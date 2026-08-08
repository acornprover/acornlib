from nat import Nat
from algebra.group import Group, has_finite_order, is_torsion_free
from order import PartialOrder, LinearOrder
from order_relation import lt_relation_is_transitive
from order import is_monotone, is_strict_monotone, is_order_embedding,
    order_embedding_is_strict_monotone, order_embedding_apply_le, order_embedding_apply_lt,
    order_embedding_apply_ge, order_embedding_apply_gt, order_embedding_reflects_le,
    order_embedding_reflects_lt, order_embedding_reflects_ge, order_embedding_reflects_gt
from order_iso import OrderIso, is_order_iso_pair, order_iso_new_map, order_iso_new_inv,
    order_iso_le_iff_le, order_iso_lt_iff_lt, order_iso_ge_iff_ge, order_iso_gt_iff_gt
from data.basic.relation_basic import antisymmetric_eq, transitive_step

/// A left-ordered group is a group with a left-invariant order.
/// This means that if `a <= b`, then `c * a <= c * b` for any c in the group.
typeclass G: LeftOrderedGroup extends Group, LinearOrder {
    /// Left multiplication preserves the order relation: if `a ≤ b`, then `c * a ≤ c * b`.
    left_invariance(a: G, b: G, c: G) {
        a <= b implies c * a <= c * b
    }
}

/// True if a is strictly less than b in the group order.
define lt[G: LeftOrderedGroup](a: G, b: G) -> Bool {
    a <= b and a != b
}

theorem lt_trans[G: LeftOrderedGroup](a: G, b: G, c: G) {
    lt(a, b) and lt(b, c) implies lt(a, c)
} by {
    if lt(a, b) and lt(b, c) {
        a <= b
        a != b
        b <= c
        transitive_step(G.lte, a, b, c)
        a <= c

        if a = c {
            c <= b
            b <= a
            antisymmetric_eq(G.lte, a, b)
            a = b
            false
        }
        a != c
        lt(a, c)
    }
}

theorem one_lt_pow[G: LeftOrderedGroup](a: G, n: Nat) {
    lt(G.1, a) and n != Nat.0 implies lt(G.1, a.pow(n))
} by {
    // Define a helper function for induction
    define f(x: Nat) -> Bool {
        lt(G.1, a.pow(x.suc))
    }
    f(Nat.0)
    forall(x: Nat) {
        if f(x) {
            a * G.1 <= a * a.pow(x.suc)
            lt(G.1, a.pow(x.suc.suc))
            f(x.suc)
        }
    }
    forall(k: Nat) { f(k) }
    if n = Nat.0 {
        false
    } else {
        let m: Nat satisfy { m.suc = n }
        f(m)
        lt(G.1, a.pow(n))
    }
}

theorem pow_lt_one[G: LeftOrderedGroup](a: G, n: Nat) {
    lt(a, G.1) and n != Nat.0 implies lt(a.pow(n), G.1)
} by {
    // Define a helper function for induction
    define f(x: Nat) -> Bool {
        lt(a.pow(x.suc), G.1)
    }
    f(Nat.0)
    forall(x: Nat) {
        if f(x) {
            a * a.pow(x.suc) <= a * G.1
            lt(a * a.pow(x.suc), a)
            f(x.suc)
        }
    }
    forall(k: Nat) { f(k) }
    if n = Nat.0 {
        false
    } else {
        let m: Nat satisfy { m.suc = n }
        f(m)
        lt(a.pow(n), G.1)
    }
}

/// An ordered group has both left and right invariance of the order under multiplication.
typeclass G: OrderedGroup extends LeftOrderedGroup {
    /// Right multiplication preserves the order relation: if `a ≤ b`, then `a * c ≤ b * c`.
    right_invariance(a: G, b: G, c: G) {
        a <= b implies a * c <= b * c
    }
}

// Ordered groups must be torsion-free.
theorem ordered_imp_torsion_free[G: OrderedGroup](g: G) {
    is_torsion_free[G]
} by {
    forall(h: G) {
        if has_finite_order(h) {
            if h != G.1 {
                // h has finite order, so there exists n != 0 with h.pow(n) = G.1
                let n: Nat satisfy { n != Nat.0 and h.pow(n) = G.1 }
                n != Nat.0
                h.pow(n) = G.1

                if lt(G.1, h) {
                    // one_lt_pow gives lt(G.1, h.pow(n))
                    lt(G.1, h.pow(n))
                    // But h.pow(n) = G.1, so lt(G.1, G.1) which is false
                    h.pow(n) != G.1
                    false
                } else {
                    // LinearOrder: h <= G.1 or G.1 <= h
                    h <= G.1 or G.1 <= h
                    // Since not lt(G.1, h), we have h <= G.1
                    // Since h != G.1 and h <= G.1, we have lt(h, G.1)
                    h <= G.1
                    lt(h, G.1)
                    // pow_lt_one gives lt(h.pow(n), G.1)
                    lt(h.pow(n), G.1)
                    // But h.pow(n) = G.1, so lt(G.1, G.1) which is false
                    h.pow(n) != G.1
                    false
                }
            }
            h = G.1
        }
        not has_finite_order(h) or h = G.1
    }
}

/// The map obtained by multiplying by a fixed element on the left.
define mul_left_map[G: LeftOrderedGroup](c: G, x: G) -> G {
    c * x
}

/// The map obtained by multiplying by a fixed element on the right.
define mul_right_map[G: OrderedGroup](c: G, x: G) -> G {
    x * c
}

/// Left multiplication by a fixed element is the corresponding product.
theorem mul_left_map_apply[G: LeftOrderedGroup](c: G, x: G) {
    mul_left_map(c, x) = c * x
}

/// Right multiplication by a fixed element is the corresponding product.
theorem mul_right_map_apply[G: OrderedGroup](c: G, x: G) {
    mul_right_map(c, x) = x * c
}

/// Left multiplication by a fixed element preserves the non-strict order.
theorem mul_left_map_is_monotone[G: LeftOrderedGroup](c: G) {
    is_monotone(mul_left_map(c))
} by {
    forall(x: G, y: G) {
        if x <= y {
            c * x <= c * y
            mul_left_map(c, x) <= mul_left_map(c, y)
        }
    }
}

/// Right multiplication by a fixed element preserves the non-strict order.
theorem mul_right_map_is_monotone[G: OrderedGroup](c: G) {
    is_monotone(mul_right_map(c))
} by {
    forall(x: G, y: G) {
        if x <= y {
            x * c <= y * c
            mul_right_map(c, x) <= mul_right_map(c, y)
        }
    }
}

/// Left multiplication by a fixed element reflects the non-strict order.
theorem mul_left_map_reflects_le[G: LeftOrderedGroup](c: G, x: G, y: G) {
    mul_left_map(c, x) <= mul_left_map(c, y) implies x <= y
} by {
    if mul_left_map(c, x) <= mul_left_map(c, y) {
        mul_left_map(c, x) = c * x
        mul_left_map(c, y) = c * y
        c.inverse * (c * x) <= c.inverse * (c * y)
        c.inverse * (c * x) = c.inverse * c * x
        c.inverse * (c * y) = c.inverse * c * y
        c.inverse * c = G.1
        G.1 * x = x
        G.1 * y = y
        x <= y
    }
}

/// Right multiplication by a fixed element reflects the non-strict order.
theorem mul_right_map_reflects_le[G: OrderedGroup](c: G, x: G, y: G) {
    mul_right_map(c, x) <= mul_right_map(c, y) implies x <= y
} by {
    if mul_right_map(c, x) <= mul_right_map(c, y) {
        mul_right_map(c, x) = x * c
        mul_right_map(c, y) = y * c
        (x * c) * c.inverse <= (y * c) * c.inverse
        x * c * c.inverse = x * (c * c.inverse)
        y * c * c.inverse = y * (c * c.inverse)
        c * c.inverse = G.1
        x * G.1 = x
        y * G.1 = y
        x <= y
    }
}

/// Left multiplication by a fixed element preserves and reflects the non-strict order.
theorem mul_left_map_is_order_embedding[G: LeftOrderedGroup](c: G) {
    is_order_embedding(mul_left_map(c))
} by {
    forall(x: G, y: G) {
        if mul_left_map(c, x) <= mul_left_map(c, y) {
            mul_left_map_reflects_le(c, x, y)
            x <= y
        }
        if x <= y {
            mul_left_map_is_monotone(c)
            is_monotone(mul_left_map(c))
            mul_left_map(c, x) <= mul_left_map(c, y)
        }
        mul_left_map(c, x) <= mul_left_map(c, y) = (x <= y)
    }
}

/// Right multiplication by a fixed element preserves and reflects the non-strict order.
theorem mul_right_map_is_order_embedding[G: OrderedGroup](c: G) {
    is_order_embedding(mul_right_map(c))
} by {
    forall(x: G, y: G) {
        if mul_right_map(c, x) <= mul_right_map(c, y) {
            mul_right_map_reflects_le(c, x, y)
            x <= y
        }
        if x <= y {
            mul_right_map_is_monotone(c)
            is_monotone(mul_right_map(c))
            mul_right_map(c, x) <= mul_right_map(c, y)
        }
        mul_right_map(c, x) <= mul_right_map(c, y) = (x <= y)
    }
}

/// Left multiplication by a fixed element preserves strict order.
theorem mul_left_map_is_strict_monotone[G: LeftOrderedGroup](c: G) {
    is_strict_monotone(mul_left_map(c))
} by {
    mul_left_map_is_order_embedding(c)
    order_embedding_is_strict_monotone(mul_left_map(c))
}

/// Right multiplication by a fixed element preserves strict order.
theorem mul_right_map_is_strict_monotone[G: OrderedGroup](c: G) {
    is_strict_monotone(mul_right_map(c))
} by {
    mul_right_map_is_order_embedding(c)
    order_embedding_is_strict_monotone(mul_right_map(c))
}

/// Left multiplication by the inverse is a left inverse of left multiplication.
theorem mul_left_map_left_inverse[G: LeftOrderedGroup](c: G, x: G) {
    mul_left_map(c.inverse, mul_left_map(c, x)) = x
} by {
    mul_left_map(c.inverse, mul_left_map(c, x)) = c.inverse * mul_left_map(c, x)
    mul_left_map(c, x) = c * x
    c.inverse * (c * x) = c.inverse * c * x
    c.inverse * c = G.1
    G.1 * x = x
    mul_left_map(c.inverse, mul_left_map(c, x)) = x
}

/// Left multiplication is a left inverse of left multiplication by the inverse.
theorem mul_left_map_right_inverse[G: LeftOrderedGroup](c: G, x: G) {
    mul_left_map(c, mul_left_map(c.inverse, x)) = x
} by {
    mul_left_map(c, mul_left_map(c.inverse, x)) = c * mul_left_map(c.inverse, x)
    mul_left_map(c.inverse, x) = c.inverse * x
    c * (c.inverse * x) = c * c.inverse * x
    c * c.inverse = G.1
    G.1 * x = x
    mul_left_map(c, mul_left_map(c.inverse, x)) = x
}

/// Right multiplication by the inverse is a left inverse of right multiplication.
theorem mul_right_map_left_inverse[G: OrderedGroup](c: G, x: G) {
    mul_right_map(c.inverse, mul_right_map(c, x)) = x
} by {
    mul_right_map(c.inverse, mul_right_map(c, x)) = mul_right_map(c, x) * c.inverse
    mul_right_map(c, x) = x * c
    (x * c) * c.inverse = x * (c * c.inverse)
    c * c.inverse = G.1
    x * G.1 = x
    mul_right_map(c.inverse, mul_right_map(c, x)) = x
}

/// Right multiplication is a left inverse of right multiplication by the inverse.
theorem mul_right_map_right_inverse[G: OrderedGroup](c: G, x: G) {
    mul_right_map(c, mul_right_map(c.inverse, x)) = x
} by {
    mul_right_map(c, mul_right_map(c.inverse, x)) = mul_right_map(c.inverse, x) * c
    mul_right_map(c.inverse, x) = x * c.inverse
    (x * c.inverse) * c = x * (c.inverse * c)
    c.inverse * c = G.1
    x * G.1 = x
    mul_right_map(c, mul_right_map(c.inverse, x)) = x
}

/// Left multiplication and multiplication by the inverse form an order isomorphism pair.
theorem mul_left_map_is_order_iso_pair[G: LeftOrderedGroup](c: G) {
    is_order_iso_pair(mul_left_map(c), mul_left_map(c.inverse))
} by {
    let f = mul_left_map(c)
    let g = mul_left_map(c.inverse)
    let left_inverse: Bool = forall(x: G) {
        g(f(x)) = x
    }
    let right_inverse: Bool = forall(y: G) {
        f(g(y)) = y
    }
    forall(x: G) {
        mul_left_map_left_inverse(c, x)
        g(f(x)) = x
    }
    left_inverse
    forall(y: G) {
        mul_left_map_right_inverse(c, y)
        f(g(y)) = y
    }
    right_inverse
    mul_left_map_is_order_embedding(c)
    mul_left_map_is_order_embedding(c.inverse)
    is_order_embedding(f)
    is_order_embedding(g)
    left_inverse and right_inverse
    left_inverse and right_inverse and is_order_embedding(f)
    left_inverse and right_inverse and is_order_embedding(f) and is_order_embedding(g)
    is_order_iso_pair(f, g) = (left_inverse and right_inverse and is_order_embedding(f) and is_order_embedding(g))
    left_inverse = forall(x: G) {
        g(f(x)) = x
    }
    right_inverse = forall(y: G) {
        f(g(y)) = y
    }
    is_order_iso_pair(f, g)
    f = mul_left_map(c)
    g = mul_left_map(c.inverse)
    is_order_iso_pair(mul_left_map(c), mul_left_map(c.inverse))
}

/// Right multiplication and multiplication by the inverse form an order isomorphism pair.
theorem mul_right_map_is_order_iso_pair[G: OrderedGroup](c: G) {
    is_order_iso_pair(mul_right_map(c), mul_right_map(c.inverse))
} by {
    let f = mul_right_map(c)
    let g = mul_right_map(c.inverse)
    let left_inverse: Bool = forall(x: G) {
        g(f(x)) = x
    }
    let right_inverse: Bool = forall(y: G) {
        f(g(y)) = y
    }
    forall(x: G) {
        mul_right_map_left_inverse(c, x)
        g(f(x)) = x
    }
    left_inverse
    forall(y: G) {
        mul_right_map_right_inverse(c, y)
        f(g(y)) = y
    }
    right_inverse
    mul_right_map_is_order_embedding(c)
    mul_right_map_is_order_embedding(c.inverse)
    is_order_embedding(f)
    is_order_embedding(g)
    left_inverse and right_inverse
    left_inverse and right_inverse and is_order_embedding(f)
    left_inverse and right_inverse and is_order_embedding(f) and is_order_embedding(g)
    is_order_iso_pair(f, g) = (left_inverse and right_inverse and is_order_embedding(f) and is_order_embedding(g))
    left_inverse = forall(x: G) {
        g(f(x)) = x
    }
    right_inverse = forall(y: G) {
        f(g(y)) = y
    }
    is_order_iso_pair(f, g)
    f = mul_right_map(c)
    g = mul_right_map(c.inverse)
    is_order_iso_pair(mul_right_map(c), mul_right_map(c.inverse))
}

/// Left multiplication by a fixed element as an order isomorphism.
let mul_left_order_iso[G: LeftOrderedGroup](c: G) -> result: OrderIso[G, G] satisfy {
    result.map = mul_left_map(c) and result.inv = mul_left_map(c.inverse)
} by {
    mul_left_map_is_order_iso_pair(c)
    let e: OrderIso[G, G] satisfy {
        OrderIso[G, G].new(mul_left_map(c), mul_left_map(c.inverse)) = Option.some(e)
    }
    order_iso_new_map(mul_left_map(c), mul_left_map(c.inverse), e)
    e.map = mul_left_map(c)
    order_iso_new_inv(mul_left_map(c), mul_left_map(c.inverse), e)
    e.inv = mul_left_map(c.inverse)
}

/// The map of the left-multiplication order isomorphism is left multiplication.
theorem mul_left_order_iso_map[G: LeftOrderedGroup](c: G) {
    mul_left_order_iso(c).map = mul_left_map(c)
} by {
    let e = mul_left_order_iso(c)
    OrderIso[G, G].new(mul_left_map(c), mul_left_map(c.inverse)) = Option.some(e)
    order_iso_new_map(mul_left_map(c), mul_left_map(c.inverse), e)
}

/// The inverse of the left-multiplication order isomorphism is left multiplication by the inverse.
theorem mul_left_order_iso_inv[G: LeftOrderedGroup](c: G) {
    mul_left_order_iso(c).inv = mul_left_map(c.inverse)
} by {
    let e = mul_left_order_iso(c)
    OrderIso[G, G].new(mul_left_map(c), mul_left_map(c.inverse)) = Option.some(e)
    order_iso_new_inv(mul_left_map(c), mul_left_map(c.inverse), e)
}

/// Right multiplication by a fixed element as an order isomorphism.
let mul_right_order_iso[G: OrderedGroup](c: G) -> result: OrderIso[G, G] satisfy {
    result.map = mul_right_map(c) and result.inv = mul_right_map(c.inverse)
} by {
    mul_right_map_is_order_iso_pair(c)
    let e: OrderIso[G, G] satisfy {
        OrderIso[G, G].new(mul_right_map(c), mul_right_map(c.inverse)) = Option.some(e)
    }
    order_iso_new_map(mul_right_map(c), mul_right_map(c.inverse), e)
    e.map = mul_right_map(c)
    order_iso_new_inv(mul_right_map(c), mul_right_map(c.inverse), e)
    e.inv = mul_right_map(c.inverse)
}

/// The map of the right-multiplication order isomorphism is right multiplication.
theorem mul_right_order_iso_map[G: OrderedGroup](c: G) {
    mul_right_order_iso(c).map = mul_right_map(c)
} by {
    let e = mul_right_order_iso(c)
    OrderIso[G, G].new(mul_right_map(c), mul_right_map(c.inverse)) = Option.some(e)
    order_iso_new_map(mul_right_map(c), mul_right_map(c.inverse), e)
}

/// The inverse of the right-multiplication order isomorphism is right multiplication by the inverse.
theorem mul_right_order_iso_inv[G: OrderedGroup](c: G) {
    mul_right_order_iso(c).inv = mul_right_map(c.inverse)
} by {
    let e = mul_right_order_iso(c)
    OrderIso[G, G].new(mul_right_map(c), mul_right_map(c.inverse)) = Option.some(e)
    order_iso_new_inv(mul_right_map(c), mul_right_map(c.inverse), e)
}

/// Multiplying by a fixed element on the left preserves a non-strict comparison.
theorem mul_le_mul_left[G: LeftOrderedGroup](a: G, b: G, c: G) {
    a <= b implies c * a <= c * b
} by {
    if a <= b {
        mul_left_map_is_order_embedding(c)
        order_embedding_apply_le(mul_left_map(c), a, b)
        mul_left_map(c, a) <= mul_left_map(c, b)
        c * a <= c * b
    }
}

/// Multiplying by a fixed element on the right preserves a non-strict comparison.
theorem mul_le_mul_right[G: OrderedGroup](a: G, b: G, c: G) {
    a <= b implies a * c <= b * c
} by {
    if a <= b {
        mul_right_map_is_order_embedding(c)
        order_embedding_apply_le(mul_right_map(c), a, b)
        mul_right_map(c, a) <= mul_right_map(c, b)
        a * c <= b * c
    }
}

/// Multiplying by a fixed element on the left preserves a strict comparison.
theorem mul_lt_mul_left[G: LeftOrderedGroup](a: G, b: G, c: G) {
    a < b implies c * a < c * b
} by {
    if a < b {
        mul_left_map_is_order_embedding(c)
        order_embedding_apply_lt(mul_left_map(c), a, b)
        mul_left_map(c, a) < mul_left_map(c, b)
        c * a < c * b
    }
}

/// Multiplying by a fixed element on the right preserves a strict comparison.
theorem mul_lt_mul_right[G: OrderedGroup](a: G, b: G, c: G) {
    a < b implies a * c < b * c
} by {
    if a < b {
        mul_right_map_is_order_embedding(c)
        order_embedding_apply_lt(mul_right_map(c), a, b)
        mul_right_map(c, a) < mul_right_map(c, b)
        a * c < b * c
    }
}

/// Multiplying by a fixed element on the left preserves a reverse non-strict comparison.
theorem mul_ge_mul_left[G: LeftOrderedGroup](a: G, b: G, c: G) {
    a >= b implies c * a >= c * b
} by {
    if a >= b {
        mul_left_map_is_order_embedding(c)
        order_embedding_apply_ge(mul_left_map(c), a, b)
        mul_left_map(c, a) >= mul_left_map(c, b)
        c * a >= c * b
    }
}

/// Multiplying by a fixed element on the right preserves a reverse non-strict comparison.
theorem mul_ge_mul_right[G: OrderedGroup](a: G, b: G, c: G) {
    a >= b implies a * c >= b * c
} by {
    if a >= b {
        mul_right_map_is_order_embedding(c)
        order_embedding_apply_ge(mul_right_map(c), a, b)
        mul_right_map(c, a) >= mul_right_map(c, b)
        a * c >= b * c
    }
}

/// Multiplying by a fixed element on the left preserves a strict reverse comparison.
theorem mul_gt_mul_left[G: LeftOrderedGroup](a: G, b: G, c: G) {
    a > b implies c * a > c * b
} by {
    if a > b {
        mul_left_map_is_order_embedding(c)
        order_embedding_apply_gt(mul_left_map(c), a, b)
        mul_left_map(c, a) > mul_left_map(c, b)
        c * a > c * b
    }
}

/// Multiplying by a fixed element on the right preserves a strict reverse comparison.
theorem mul_gt_mul_right[G: OrderedGroup](a: G, b: G, c: G) {
    a > b implies a * c > b * c
} by {
    if a > b {
        mul_right_map_is_order_embedding(c)
        order_embedding_apply_gt(mul_right_map(c), a, b)
        mul_right_map(c, a) > mul_right_map(c, b)
        a * c > b * c
    }
}

/// A non-strict comparison after multiplying on the left reflects to the original comparison.
theorem le_of_mul_le_mul_left[G: LeftOrderedGroup](a: G, b: G, c: G) {
    c * a <= c * b implies a <= b
} by {
    if c * a <= c * b {
        mul_left_map_reflects_le(c, a, b)
        a <= b
    }
}

/// A non-strict comparison after multiplying on the right reflects to the original comparison.
theorem le_of_mul_le_mul_right[G: OrderedGroup](a: G, b: G, c: G) {
    a * c <= b * c implies a <= b
} by {
    if a * c <= b * c {
        mul_right_map_reflects_le(c, a, b)
        a <= b
    }
}

/// A strict comparison after multiplying on the left reflects to the original comparison.
theorem lt_of_mul_lt_mul_left[G: LeftOrderedGroup](a: G, b: G, c: G) {
    c * a < c * b implies a < b
} by {
    if c * a < c * b {
        mul_left_map_is_order_embedding(c)
        order_embedding_reflects_lt(mul_left_map(c), a, b)
        a < b
    }
}

/// A strict comparison after multiplying on the right reflects to the original comparison.
theorem lt_of_mul_lt_mul_right[G: OrderedGroup](a: G, b: G, c: G) {
    a * c < b * c implies a < b
} by {
    if a * c < b * c {
        mul_right_map_is_order_embedding(c)
        order_embedding_reflects_lt(mul_right_map(c), a, b)
        a < b
    }
}

/// A reverse non-strict comparison after multiplying on the left reflects to the original comparison.
theorem ge_of_mul_ge_mul_left[G: LeftOrderedGroup](a: G, b: G, c: G) {
    c * a >= c * b implies a >= b
} by {
    if c * a >= c * b {
        mul_left_map_is_order_embedding(c)
        order_embedding_reflects_ge(mul_left_map(c), a, b)
        a >= b
    }
}

/// A reverse non-strict comparison after multiplying on the right reflects to the original comparison.
theorem ge_of_mul_ge_mul_right[G: OrderedGroup](a: G, b: G, c: G) {
    a * c >= b * c implies a >= b
} by {
    if a * c >= b * c {
        mul_right_map_is_order_embedding(c)
        order_embedding_reflects_ge(mul_right_map(c), a, b)
        a >= b
    }
}

/// A strict reverse comparison after multiplying on the left reflects to the original comparison.
theorem gt_of_mul_gt_mul_left[G: LeftOrderedGroup](a: G, b: G, c: G) {
    c * a > c * b implies a > b
} by {
    if c * a > c * b {
        mul_left_map_is_order_embedding(c)
        order_embedding_reflects_gt(mul_left_map(c), a, b)
        a > b
    }
}

/// A strict reverse comparison after multiplying on the right reflects to the original comparison.
theorem gt_of_mul_gt_mul_right[G: OrderedGroup](a: G, b: G, c: G) {
    a * c > b * c implies a > b
} by {
    if a * c > b * c {
        mul_right_map_is_order_embedding(c)
        order_embedding_reflects_gt(mul_right_map(c), a, b)
        a > b
    }
}

/// Multiplying by a fixed element on the left preserves and reflects a non-strict comparison.
theorem mul_le_mul_left_iff[G: LeftOrderedGroup](a: G, b: G, c: G) {
    c * a <= c * b = (a <= b)
} by {
    let e = mul_left_order_iso(c)
    mul_left_order_iso_map(c)
    order_iso_le_iff_le(e, a, b)
    e.map(a) <= e.map(b) = (a <= b)
    e.map = mul_left_map(c)
    e.map(a) = mul_left_map(c, a)
    e.map(b) = mul_left_map(c, b)
    c * a <= c * b = (a <= b)
}

/// Multiplying by a fixed element on the right preserves and reflects a non-strict comparison.
theorem mul_le_mul_right_iff[G: OrderedGroup](a: G, b: G, c: G) {
    a * c <= b * c = (a <= b)
} by {
    let e = mul_right_order_iso(c)
    mul_right_order_iso_map(c)
    order_iso_le_iff_le(e, a, b)
    e.map(a) <= e.map(b) = (a <= b)
    e.map = mul_right_map(c)
    e.map(a) = mul_right_map(c, a)
    e.map(b) = mul_right_map(c, b)
    a * c <= b * c = (a <= b)
}

/// Multiplying by a fixed element on the left preserves and reflects a strict comparison.
theorem mul_lt_mul_left_iff[G: LeftOrderedGroup](a: G, b: G, c: G) {
    c * a < c * b = (a < b)
} by {
    let e = mul_left_order_iso(c)
    mul_left_order_iso_map(c)
    order_iso_lt_iff_lt(e, a, b)
    e.map(a) < e.map(b) = (a < b)
    e.map = mul_left_map(c)
    e.map(a) = mul_left_map(c, a)
    e.map(b) = mul_left_map(c, b)
    c * a < c * b = (a < b)
}

/// Multiplying by a fixed element on the right preserves and reflects a strict comparison.
theorem mul_lt_mul_right_iff[G: OrderedGroup](a: G, b: G, c: G) {
    a * c < b * c = (a < b)
} by {
    let e = mul_right_order_iso(c)
    mul_right_order_iso_map(c)
    order_iso_lt_iff_lt(e, a, b)
    e.map(a) < e.map(b) = (a < b)
    e.map = mul_right_map(c)
    e.map(a) = mul_right_map(c, a)
    e.map(b) = mul_right_map(c, b)
    a * c < b * c = (a < b)
}

/// Multiplying by a fixed element on the left preserves and reflects a reverse non-strict comparison.
theorem mul_ge_mul_left_iff[G: LeftOrderedGroup](a: G, b: G, c: G) {
    c * a >= c * b = (a >= b)
} by {
    let e = mul_left_order_iso(c)
    mul_left_order_iso_map(c)
    order_iso_ge_iff_ge(e, a, b)
    e.map(a) >= e.map(b) = (a >= b)
    e.map = mul_left_map(c)
    e.map(a) = mul_left_map(c, a)
    e.map(b) = mul_left_map(c, b)
    c * a >= c * b = (a >= b)
}

/// Multiplying by a fixed element on the right preserves and reflects a reverse non-strict comparison.
theorem mul_ge_mul_right_iff[G: OrderedGroup](a: G, b: G, c: G) {
    a * c >= b * c = (a >= b)
} by {
    let e = mul_right_order_iso(c)
    mul_right_order_iso_map(c)
    order_iso_ge_iff_ge(e, a, b)
    e.map(a) >= e.map(b) = (a >= b)
    e.map = mul_right_map(c)
    e.map(a) = mul_right_map(c, a)
    e.map(b) = mul_right_map(c, b)
    a * c >= b * c = (a >= b)
}

/// Multiplying by a fixed element on the left preserves and reflects a strict reverse comparison.
theorem mul_gt_mul_left_iff[G: LeftOrderedGroup](a: G, b: G, c: G) {
    c * a > c * b = (a > b)
} by {
    let e = mul_left_order_iso(c)
    mul_left_order_iso_map(c)
    order_iso_gt_iff_gt(e, a, b)
    e.map(a) > e.map(b) = (a > b)
    e.map = mul_left_map(c)
    e.map(a) = mul_left_map(c, a)
    e.map(b) = mul_left_map(c, b)
    c * a > c * b = (a > b)
}

/// Multiplying by a fixed element on the right preserves and reflects a strict reverse comparison.
theorem mul_gt_mul_right_iff[G: OrderedGroup](a: G, b: G, c: G) {
    a * c > b * c = (a > b)
} by {
    let e = mul_right_order_iso(c)
    mul_right_order_iso_map(c)
    order_iso_gt_iff_gt(e, a, b)
    e.map(a) > e.map(b) = (a > b)
    e.map = mul_right_map(c)
    e.map(a) = mul_right_map(c, a)
    e.map(b) = mul_right_map(c, b)
    a * c > b * c = (a > b)
}
