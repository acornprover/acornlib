/// Contravariant maps on prime spectra and distinguished basic opens.

from comm_ring import CommRing
from algebra.ring.ring_hom import RingHom
from algebra.ring.ideal import Ideal, bundled_ideal_subset, ideal_preimage,
    prime_ideal_preimage, prime_ideal_preimage_contains_eq
from prime_ideal_preimage_as_ideal import prime_ideal_preimage_as_ideal_eq
from algebra.ring.ring_ideal_hom_image import ideal_image, ideal_image_subset_iff_subset_preimage
from algebra.ring.spec import Spec, spec_of_prime_point, spec_vanishing_contains_eq,
    spec_basic_open_contains_eq, spec_zariski_closed, spec_zariski_open,
    spec_zariski_open_basic
from data.basic.set import Set, set_preimage, set_preimage_contains_image, set_preimage_compl,
    set_ext

/// The inverse-image map on prime spectra induced by a ring homomorphism.
define spec_comap[R: CommRing, S: CommRing](f: RingHom[R, S], p: Spec[S]) -> Spec[R] {
    Spec[R].of_prime(prime_ideal_preimage(f, p.point))
}

/// The prime ideal underlying `spec_comap` is the inverse-image prime ideal.
theorem spec_comap_point[R: CommRing, S: CommRing](f: RingHom[R, S], p: Spec[S]) {
    spec_comap(f, p).point = prime_ideal_preimage(f, p.point)
} by {
    spec_comap(f, p) = Spec[R].of_prime(prime_ideal_preimage(f, p.point))
    spec_of_prime_point(prime_ideal_preimage(f, p.point))
}

/// Membership in the inverse-image prime is membership after applying the homomorphism.
theorem spec_comap_contains_eq[R: CommRing, S: CommRing](f: RingHom[R, S], p: Spec[S], x: R) {
    spec_comap(f, p).point.contains(x) = p.point.contains(f.hom(x))
} by {
    spec_comap_point(f, p)
    prime_ideal_preimage_contains_eq(f, p.point, x)
}

/// The inverse image of a distinguished basic open is the distinguished basic open
/// at the image of its defining element.
theorem spec_comap_preimage_basic_open[R: CommRing, S: CommRing](f: RingHom[R, S], a: R) {
    set_preimage(spec_comap(f), Spec[R].basic_open(a)) = Spec[S].basic_open(f.hom(a))
} by {
    forall(p: Spec[S]) {
        set_preimage_contains_image(spec_comap(f), Spec[R].basic_open(a), p)
        set_preimage(spec_comap(f), Spec[R].basic_open(a)).contains(p) =
            Spec[R].basic_open(a).contains(spec_comap(f, p))
        spec_basic_open_contains_eq(a, spec_comap(f, p))
        Spec[R].basic_open(a).contains(spec_comap(f, p)) =
            not spec_comap(f, p).point.contains(a)
        spec_comap_contains_eq(f, p, a)
        spec_comap(f, p).point.contains(a) = p.point.contains(f.hom(a))
        not spec_comap(f, p).point.contains(a) = not p.point.contains(f.hom(a))
        spec_basic_open_contains_eq(f.hom(a), p)
        Spec[S].basic_open(f.hom(a)).contains(p) = not p.point.contains(f.hom(a))
        if set_preimage(spec_comap(f), Spec[R].basic_open(a)).contains(p) {
            Spec[R].basic_open(a).contains(spec_comap(f, p))
            not spec_comap(f, p).point.contains(a)
            not p.point.contains(f.hom(a))
            Spec[S].basic_open(f.hom(a)).contains(p)
        }
        if Spec[S].basic_open(f.hom(a)).contains(p) {
            not p.point.contains(f.hom(a))
            not spec_comap(f, p).point.contains(a)
            Spec[R].basic_open(a).contains(spec_comap(f, p))
            set_preimage(spec_comap(f), Spec[R].basic_open(a)).contains(p)
        }
        set_preimage(spec_comap(f), Spec[R].basic_open(a)).contains(p) =
            Spec[S].basic_open(f.hom(a)).contains(p)
    }
    set_ext(set_preimage(spec_comap(f), Spec[R].basic_open(a)), Spec[S].basic_open(f.hom(a)))
}

/// The inverse image of a vanishing set under `Spec(S) -> Spec(R)` is the
/// vanishing set of the image ideal generated by the mapped source ideal.
theorem spec_comap_preimage_vanishing[R: CommRing, S: CommRing](f: RingHom[R, S], i: Ideal[R]) {
    set_preimage(spec_comap(f), Spec[R].vanishing(i)) =
        Spec[S].vanishing(ideal_image(f, i))
} by {
    forall(p: Spec[S]) {
        set_preimage_contains_image(spec_comap(f), Spec[R].vanishing(i), p)
        set_preimage(spec_comap(f), Spec[R].vanishing(i)).contains(p) =
            Spec[R].vanishing(i).contains(spec_comap(f, p))
        spec_vanishing_contains_eq(i, spec_comap(f, p))
        spec_vanishing_contains_eq(ideal_image(f, i), p)
        prime_ideal_preimage_as_ideal_eq(f, p.point)
        spec_comap_point(f, p)
        spec_comap(f, p).point.as_ideal = ideal_preimage(f, p.point.as_ideal)
        ideal_image_subset_iff_subset_preimage(f, i, p.point.as_ideal)
        bundled_ideal_subset(ideal_image(f, i), p.point.as_ideal) =
            bundled_ideal_subset(i, ideal_preimage(f, p.point.as_ideal))
        if set_preimage(spec_comap(f), Spec[R].vanishing(i)).contains(p) {
            Spec[R].vanishing(i).contains(spec_comap(f, p))
            bundled_ideal_subset(i, spec_comap(f, p).point.as_ideal)
            bundled_ideal_subset(i, ideal_preimage(f, p.point.as_ideal))
            bundled_ideal_subset(ideal_image(f, i), p.point.as_ideal)
            Spec[S].vanishing(ideal_image(f, i)).contains(p)
        }
        if Spec[S].vanishing(ideal_image(f, i)).contains(p) {
            bundled_ideal_subset(ideal_image(f, i), p.point.as_ideal)
            bundled_ideal_subset(i, ideal_preimage(f, p.point.as_ideal))
            bundled_ideal_subset(i, spec_comap(f, p).point.as_ideal)
            Spec[R].vanishing(i).contains(spec_comap(f, p))
            set_preimage(spec_comap(f), Spec[R].vanishing(i)).contains(p)
        }
        set_preimage(spec_comap(f), Spec[R].vanishing(i)).contains(p) =
            Spec[S].vanishing(ideal_image(f, i)).contains(p)
    }
    set_ext(set_preimage(spec_comap(f), Spec[R].vanishing(i)),
        Spec[S].vanishing(ideal_image(f, i)))
}

/// A Zariski-closed set has Zariski-closed inverse image under comap.
theorem spec_comap_preimage_zariski_closed[R: CommRing, S: CommRing](
    f: RingHom[R, S], c: Set[Spec[R]]
) {
    spec_zariski_closed(c) implies spec_zariski_closed(set_preimage(spec_comap(f), c))
} by {
    if spec_zariski_closed(c) {
        let i: Ideal[R] satisfy { c = Spec[R].vanishing(i) }
        spec_comap_preimage_vanishing(f, i)
        set_preimage(spec_comap(f), c) = set_preimage(spec_comap(f), Spec[R].vanishing(i))
        set_preimage(spec_comap(f), c) = Spec[S].vanishing(ideal_image(f, i))
        exists(j: Ideal[S]) {
            set_preimage(spec_comap(f), c) = Spec[S].vanishing(j)
        }
    }
}

/// A basic open has Zariski-open inverse image under comap.
theorem spec_comap_preimage_basic_open_zariski_open[R: CommRing, S: CommRing](
    f: RingHom[R, S], a: R
) {
    spec_zariski_open(set_preimage(spec_comap(f), Spec[R].basic_open(a)))
} by {
    spec_comap_preimage_basic_open(f, a)
    set_preimage(spec_comap(f), Spec[R].basic_open(a)) = Spec[S].basic_open(f.hom(a))
    spec_zariski_open_basic(f.hom(a))
}

/// A Zariski-open set has Zariski-open inverse image under comap.
theorem spec_comap_preimage_zariski_open[R: CommRing, S: CommRing](
    f: RingHom[R, S], u: Set[Spec[R]]
) {
    spec_zariski_open(u) implies spec_zariski_open(set_preimage(spec_comap(f), u))
} by {
    if spec_zariski_open(u) {
        let i: Ideal[R] satisfy { u = Spec[R].vanishing(i).c }
        spec_comap_preimage_vanishing(f, i)
        set_preimage(spec_comap(f), Spec[R].vanishing(i)) =
            Spec[S].vanishing(ideal_image(f, i))
        forall(p: Spec[S]) {
            set_preimage_contains_image(spec_comap(f), u, p)
            set_preimage_contains_image(spec_comap(f), Spec[R].vanishing(i).c, p)
            set_preimage(spec_comap(f), u).contains(p) = u.contains(spec_comap(f, p))
            u.contains(spec_comap(f, p)) = Spec[R].vanishing(i).c.contains(spec_comap(f, p))
            set_preimage(spec_comap(f), Spec[R].vanishing(i).c).contains(p) =
                Spec[R].vanishing(i).c.contains(spec_comap(f, p))
            set_preimage(spec_comap(f), u).contains(p) =
                set_preimage(spec_comap(f), Spec[R].vanishing(i).c).contains(p)
        }
        set_ext(set_preimage(spec_comap(f), u),
            set_preimage(spec_comap(f), Spec[R].vanishing(i).c))
        set_preimage(spec_comap(f), u) = set_preimage(spec_comap(f), Spec[R].vanishing(i).c)
        set_preimage_compl(spec_comap(f), Spec[R].vanishing(i))
        set_preimage(spec_comap(f), Spec[R].vanishing(i).c) =
            set_preimage(spec_comap(f), Spec[R].vanishing(i)).c
        set_preimage(spec_comap(f), u) = Spec[S].vanishing(ideal_image(f, i)).c
        exists(j: Ideal[S]) {
            set_preimage(spec_comap(f), u) = Spec[S].vanishing(j).c
        }
    }
}
