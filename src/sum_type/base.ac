from data.basic.functions import compose, identity_fn

/// A value belonging to one of two types.
inductive Sum[T, U] {
    /// A value from the left type.
    inl(T)

    /// A value from the right type.
    inr(U)
}

/// The left injection into a sum type.
define sum_inl[T, U](x: T) -> Sum[T, U] {
    Sum.inl[T, U](x)
}

/// The right injection into a sum type.
define sum_inr[T, U](y: U) -> Sum[T, U] {
    Sum.inr[T, U](y)
}

/// The named left injection is the left constructor.
theorem sum_inl_eq[T, U](x: T) {
    sum_inl[T, U](x) = Sum.inl[T, U](x)
}

/// The named right injection is the right constructor.
theorem sum_inr_eq[T, U](y: U) {
    sum_inr[T, U](y) = Sum.inr[T, U](y)
}

/// Case analysis for a sum value.
define sum_cases[T, U, V](s: Sum[T, U], left: T -> V, right: U -> V) -> V {
    match s {
        Sum.inl(x) {
            left(x)
        }
        Sum.inr(y) {
            right(y)
        }
    }
}

/// The image of a sum value under maps on both sides.
define sum_map[T, U, V, W](s: Sum[T, U], left: T -> V, right: U -> W) -> Sum[V, W] {
    match s {
        Sum.inl(x) {
            Sum.inl[V, W](left(x))
        }
        Sum.inr(y) {
            Sum.inr[V, W](right(y))
        }
    }
}

/// The image of a sum value under a map on the left side.
define sum_map_left[T, U, V](s: Sum[T, U], left: T -> V) -> Sum[V, U] {
    sum_map(s, left, identity_fn[U])
}

/// The image of a sum value under a map on the right side.
define sum_map_right[T, U, W](s: Sum[T, U], right: U -> W) -> Sum[T, W] {
    sum_map(s, identity_fn[T], right)
}

/// The same sum value with the two sides interchanged.
define sum_swap[T, U](s: Sum[T, U]) -> Sum[U, T] {
    match s {
        Sum.inl(x) {
            Sum.inr[U, T](x)
        }
        Sum.inr(y) {
            Sum.inl[U, T](y)
        }
    }
}

/// True if the sum value lies on the left side.
define sum_is_left[T, U](s: Sum[T, U]) -> Bool {
    match s {
        Sum.inl(x) {
            true
        }
        Sum.inr(y) {
            false
        }
    }
}

/// True if the sum value lies on the right side.
define sum_is_right[T, U](s: Sum[T, U]) -> Bool {
    match s {
        Sum.inl(x) {
            false
        }
        Sum.inr(y) {
            true
        }
    }
}

attributes Sum[T, U] {
    /// True if the value lies on the left side.
    let is_left: Sum[T, U] -> Bool = sum_is_left

    /// True if the value lies on the right side.
    let is_right: Sum[T, U] -> Bool = sum_is_right

    /// Case analysis for this sum value.
    define cases[V](self, left: T -> V, right: U -> V) -> V {
        sum_cases(self, left, right)
    }

    /// Applies maps to both sides.
    define map[V, W](self, left: T -> V, right: U -> W) -> Sum[V, W] {
        sum_map(self, left, right)
    }

    /// Applies a map to the left side.
    define map_left[V](self, left: T -> V) -> Sum[V, U] {
        sum_map_left(self, left)
    }

    /// Applies a map to the right side.
    define map_right[W](self, right: U -> W) -> Sum[T, W] {
        sum_map_right(self, right)
    }

    /// Interchanges the two sides.
    define swap(self) -> Sum[U, T] {
        sum_swap(self)
    }
}

/// Case analysis on a left injection applies the left branch.
theorem sum_cases_inl[T, U, V](x: T, left: T -> V, right: U -> V) {
    sum_cases(Sum.inl[T, U](x), left, right) = left(x)
}

/// Case analysis on a right injection applies the right branch.
theorem sum_cases_inr[T, U, V](y: U, left: T -> V, right: U -> V) {
    sum_cases(Sum.inr[T, U](y), left, right) = right(y)
}

/// Case analysis on a named left injection applies the left branch.
theorem sum_cases_sum_inl[T, U, V](x: T, left: T -> V, right: U -> V) {
    sum_cases(sum_inl[T, U](x), left, right) = left(x)
} by {
    sum_inl_eq[T, U](x)
    sum_cases_inl(x, left, right)
}

/// Case analysis on a named right injection applies the right branch.
theorem sum_cases_sum_inr[T, U, V](y: U, left: T -> V, right: U -> V) {
    sum_cases(sum_inr[T, U](y), left, right) = right(y)
} by {
    sum_inr_eq[T, U](y)
    sum_cases_inr(y, left, right)
}

/// Mapping a left injection applies the left map.
theorem sum_map_inl[T, U, V, W](x: T, left: T -> V, right: U -> W) {
    sum_map(Sum.inl[T, U](x), left, right) = Sum.inl[V, W](left(x))
}

/// Mapping a right injection applies the right map.
theorem sum_map_inr[T, U, V, W](y: U, left: T -> V, right: U -> W) {
    sum_map(Sum.inr[T, U](y), left, right) = Sum.inr[V, W](right(y))
}

/// Mapping the left side of a left injection applies the map.
theorem sum_map_left_inl[T, U, V](x: T, left: T -> V) {
    sum_map_left(Sum.inl[T, U](x), left) = Sum.inl[V, U](left(x))
}

/// Mapping the left side of a right injection leaves the value on the right.
theorem sum_map_left_inr[T, U, V](y: U, left: T -> V) {
    sum_map_left(Sum.inr[T, U](y), left) = Sum.inr[V, U](y)
}

/// Mapping the right side of a left injection leaves the value on the left.
theorem sum_map_right_inl[T, U, W](x: T, right: U -> W) {
    sum_map_right(Sum.inl[T, U](x), right) = Sum.inl[T, W](x)
}

/// Mapping the right side of a right injection applies the map.
theorem sum_map_right_inr[T, U, W](y: U, right: U -> W) {
    sum_map_right(Sum.inr[T, U](y), right) = Sum.inr[T, W](right(y))
}

/// A left injection lies on the left side.
theorem sum_is_left_inl[T, U](x: T) {
    sum_is_left(Sum.inl[T, U](x)) = true
}

/// A right injection does not lie on the left side.
theorem sum_is_left_inr[T, U](y: U) {
    sum_is_left(Sum.inr[T, U](y)) = false
}

/// A left injection does not lie on the right side.
theorem sum_is_right_inl[T, U](x: T) {
    sum_is_right(Sum.inl[T, U](x)) = false
}

/// A right injection lies on the right side.
theorem sum_is_right_inr[T, U](y: U) {
    sum_is_right(Sum.inr[T, U](y)) = true
}

/// Mapping both sides preserves whether a sum value lies on the left side.
theorem sum_is_left_map[T, U, V, W](s: Sum[T, U], left: T -> V, right: U -> W) {
    sum_is_left(sum_map(s, left, right)) = sum_is_left(s)
} by {
    match s {
        Sum.inl(x) {
            sum_map_inl(x, left, right)
            sum_map(s, left, right) = Sum.inl[V, W](left(x))
            sum_is_left_inl[V, W](left(x))
            sum_is_left(sum_map(s, left, right)) = true
            sum_is_left_inl[T, U](x)
            sum_is_left(s) = true
            sum_is_left(sum_map(s, left, right)) = sum_is_left(s)
        }
        Sum.inr(y) {
            sum_map_inr(y, left, right)
            sum_map(s, left, right) = Sum.inr[V, W](right(y))
            sum_is_left_inr[V, W](right(y))
            sum_is_left(sum_map(s, left, right)) = false
            sum_is_left_inr[T, U](y)
            sum_is_left(s) = false
            sum_is_left(sum_map(s, left, right)) = sum_is_left(s)
        }
    }
}

/// Mapping both sides preserves whether a sum value lies on the right side.
theorem sum_is_right_map[T, U, V, W](s: Sum[T, U], left: T -> V, right: U -> W) {
    sum_is_right(sum_map(s, left, right)) = sum_is_right(s)
} by {
    match s {
        Sum.inl(x) {
            sum_map_inl(x, left, right)
            sum_map(s, left, right) = Sum.inl[V, W](left(x))
            sum_is_right_inl[V, W](left(x))
            sum_is_right(sum_map(s, left, right)) = false
            sum_is_right_inl[T, U](x)
            sum_is_right(s) = false
            sum_is_right(sum_map(s, left, right)) = sum_is_right(s)
        }
        Sum.inr(y) {
            sum_map_inr(y, left, right)
            sum_map(s, left, right) = Sum.inr[V, W](right(y))
            sum_is_right_inr[V, W](right(y))
            sum_is_right(sum_map(s, left, right)) = true
            sum_is_right_inr[T, U](y)
            sum_is_right(s) = true
            sum_is_right(sum_map(s, left, right)) = sum_is_right(s)
        }
    }
}

/// Mapping both sides by identity gives the original sum value.
theorem sum_map_identity[T, U](s: Sum[T, U]) {
    sum_map(s, identity_fn[T], identity_fn[U]) = s
} by {
    match s {
        Sum.inl(x) {
            sum_map_inl(x, identity_fn[T], identity_fn[U])
            identity_fn[T](x) = x
            sum_map(s, identity_fn[T], identity_fn[U]) = Sum.inl[T, U](x)
            sum_map(s, identity_fn[T], identity_fn[U]) = s
        }
        Sum.inr(y) {
            sum_map_inr(y, identity_fn[T], identity_fn[U])
            identity_fn[U](y) = y
            sum_map(s, identity_fn[T], identity_fn[U]) = Sum.inr[T, U](y)
            sum_map(s, identity_fn[T], identity_fn[U]) = s
        }
    }
}

/// Sum maps compose on each side.
theorem sum_map_comp[A, B, C, D, E, F](f: C -> E, g: A -> C, h: D -> F, i: B -> D,
    s: Sum[A, B]) {
    sum_map(s, compose(f, g), compose(h, i)) = sum_map(sum_map(s, g, i), f, h)
} by {
    match s {
        Sum.inl(x) {
            sum_map_inl(x, compose(f, g), compose(h, i))
            compose(f, g, x) = f(g(x))
            sum_map(s, compose(f, g), compose(h, i)) = Sum.inl[E, F](f(g(x)))
            sum_map_inl(x, g, i)
            sum_map(s, g, i) = Sum.inl[C, D](g(x))
            sum_map_inl(g(x), f, h)
            sum_map(sum_map(s, g, i), f, h) = Sum.inl[E, F](f(g(x)))
            sum_map(s, compose(f, g), compose(h, i)) = sum_map(sum_map(s, g, i), f, h)
        }
        Sum.inr(y) {
            sum_map_inr(y, compose(f, g), compose(h, i))
            compose(h, i, y) = h(i(y))
            sum_map(s, compose(f, g), compose(h, i)) = Sum.inr[E, F](h(i(y)))
            sum_map_inr(y, g, i)
            sum_map(s, g, i) = Sum.inr[C, D](i(y))
            sum_map_inr(i(y), f, h)
            sum_map(sum_map(s, g, i), f, h) = Sum.inr[E, F](h(i(y)))
            sum_map(s, compose(f, g), compose(h, i)) = sum_map(sum_map(s, g, i), f, h)
        }
    }
}

/// Case analysis after mapping composes the branches with the side maps.
theorem sum_cases_map[T, U, V, W, X](s: Sum[T, U], left: T -> V, right: U -> W,
    out_left: V -> X, out_right: W -> X) {
    sum_cases(sum_map(s, left, right), out_left, out_right) =
        sum_cases(s, compose(out_left, left), compose(out_right, right))
} by {
    match s {
        Sum.inl(x) {
            sum_map_inl(x, left, right)
            sum_map(s, left, right) = Sum.inl[V, W](left(x))
            sum_cases_inl(left(x), out_left, out_right)
            sum_cases(sum_map(s, left, right), out_left, out_right) = out_left(left(x))
            sum_cases_inl(x, compose(out_left, left), compose(out_right, right))
            compose(out_left, left, x) = out_left(left(x))
            sum_cases(s, compose(out_left, left), compose(out_right, right)) = out_left(left(x))
            sum_cases(sum_map(s, left, right), out_left, out_right) =
                sum_cases(s, compose(out_left, left), compose(out_right, right))
        }
        Sum.inr(y) {
            sum_map_inr(y, left, right)
            sum_map(s, left, right) = Sum.inr[V, W](right(y))
            sum_cases_inr(right(y), out_left, out_right)
            sum_cases(sum_map(s, left, right), out_left, out_right) = out_right(right(y))
            sum_cases_inr(y, compose(out_left, left), compose(out_right, right))
            compose(out_right, right, y) = out_right(right(y))
            sum_cases(s, compose(out_left, left), compose(out_right, right)) = out_right(right(y))
            sum_cases(sum_map(s, left, right), out_left, out_right) =
                sum_cases(s, compose(out_left, left), compose(out_right, right))
        }
    }
}

/// Mapping the left side by identity gives the original sum value.
theorem sum_map_left_identity[T, U](s: Sum[T, U]) {
    sum_map_left(s, identity_fn[T]) = s
} by {
    sum_map_left(s, identity_fn[T]) = sum_map(s, identity_fn[T], identity_fn[U])
    sum_map_identity(s)
}

/// Mapping the right side by identity gives the original sum value.
theorem sum_map_right_identity[T, U](s: Sum[T, U]) {
    sum_map_right(s, identity_fn[U]) = s
} by {
    sum_map_right(s, identity_fn[U]) = sum_map(s, identity_fn[T], identity_fn[U])
    sum_map_identity(s)
}

/// Maps on the left side compose as functions do.
theorem sum_map_left_comp[A, B, C, D](f: C -> D, g: A -> C, s: Sum[A, B]) {
    sum_map_left(s, compose(f, g)) = sum_map_left(sum_map_left(s, g), f)
} by {
    match s {
        Sum.inl(x) {
            sum_map_left_inl[A, B, D](x, compose(f, g))
            compose(f, g, x) = f(g(x))
            sum_map_left(s, compose(f, g)) = Sum.inl[D, B](f(g(x)))
            sum_map_left_inl[A, B, C](x, g)
            sum_map_left(s, g) = Sum.inl[C, B](g(x))
            sum_map_left_inl[C, B, D](g(x), f)
            sum_map_left(sum_map_left(s, g), f) = Sum.inl[D, B](f(g(x)))
            sum_map_left(s, compose(f, g)) = sum_map_left(sum_map_left(s, g), f)
        }
        Sum.inr(y) {
            sum_map_left_inr[A, B, D](y, compose(f, g))
            sum_map_left(s, compose(f, g)) = Sum.inr[D, B](y)
            sum_map_left_inr[A, B, C](y, g)
            sum_map_left(s, g) = Sum.inr[C, B](y)
            sum_map_left_inr[C, B, D](y, f)
            sum_map_left(sum_map_left(s, g), f) = Sum.inr[D, B](y)
            sum_map_left(s, compose(f, g)) = sum_map_left(sum_map_left(s, g), f)
        }
    }
}

/// Maps on the right side compose as functions do.
theorem sum_map_right_comp[A, B, C, D](f: C -> D, g: B -> C, s: Sum[A, B]) {
    sum_map_right(s, compose(f, g)) = sum_map_right(sum_map_right(s, g), f)
} by {
    match s {
        Sum.inl(x) {
            sum_map_right_inl[A, B, D](x, compose(f, g))
            sum_map_right(s, compose(f, g)) = Sum.inl[A, D](x)
            sum_map_right_inl[A, B, C](x, g)
            sum_map_right(s, g) = Sum.inl[A, C](x)
            sum_map_right_inl[A, C, D](x, f)
            sum_map_right(sum_map_right(s, g), f) = Sum.inl[A, D](x)
            sum_map_right(s, compose(f, g)) = sum_map_right(sum_map_right(s, g), f)
        }
        Sum.inr(y) {
            sum_map_right_inr[A, B, D](y, compose(f, g))
            compose(f, g, y) = f(g(y))
            sum_map_right(s, compose(f, g)) = Sum.inr[A, D](f(g(y)))
            sum_map_right_inr[A, B, C](y, g)
            sum_map_right(s, g) = Sum.inr[A, C](g(y))
            sum_map_right_inr[A, C, D](g(y), f)
            sum_map_right(sum_map_right(s, g), f) = Sum.inr[A, D](f(g(y)))
            sum_map_right(s, compose(f, g)) = sum_map_right(sum_map_right(s, g), f)
        }
    }
}

/// Swapping a left injection gives a right injection.
theorem sum_swap_inl[T, U](x: T) {
    sum_swap(Sum.inl[T, U](x)) = Sum.inr[U, T](x)
}

/// Swapping a right injection gives a left injection.
theorem sum_swap_inr[T, U](y: U) {
    sum_swap(Sum.inr[T, U](y)) = Sum.inl[U, T](y)
}

/// Swapping both sides after mapping is mapping after swapping with the side maps interchanged.
theorem sum_swap_map[T, U, V, W](s: Sum[T, U], left: T -> V, right: U -> W) {
    sum_swap(sum_map(s, left, right)) = sum_map(sum_swap(s), right, left)
} by {
    match s {
        Sum.inl(x) {
            sum_map_inl(x, left, right)
            sum_map(s, left, right) = Sum.inl[V, W](left(x))
            sum_swap_inl[V, W](left(x))
            sum_swap(sum_map(s, left, right)) = Sum.inr[W, V](left(x))
            sum_swap_inl[T, U](x)
            sum_swap(s) = Sum.inr[U, T](x)
            sum_map_inr[U, T, W, V](x, right, left)
            sum_map(sum_swap(s), right, left) = Sum.inr[W, V](left(x))
            sum_swap(sum_map(s, left, right)) = sum_map(sum_swap(s), right, left)
        }
        Sum.inr(y) {
            sum_map_inr(y, left, right)
            sum_map(s, left, right) = Sum.inr[V, W](right(y))
            sum_swap_inr[V, W](right(y))
            sum_swap(sum_map(s, left, right)) = Sum.inl[W, V](right(y))
            sum_swap_inr[T, U](y)
            sum_swap(s) = Sum.inl[U, T](y)
            sum_map_inl[U, T, W, V](y, right, left)
            sum_map(sum_swap(s), right, left) = Sum.inl[W, V](right(y))
            sum_swap(sum_map(s, left, right)) = sum_map(sum_swap(s), right, left)
        }
    }
}

/// Swapping interchanges the left-side predicate with the right-side predicate.
theorem sum_is_left_swap[T, U](s: Sum[T, U]) {
    sum_is_left(sum_swap(s)) = sum_is_right(s)
} by {
    match s {
        Sum.inl(x) {
            sum_swap_inl[T, U](x)
            sum_swap(s) = Sum.inr[U, T](x)
            sum_is_left_inr[U, T](x)
            sum_is_left(sum_swap(s)) = false
            sum_is_right_inl[T, U](x)
            sum_is_right(s) = false
            sum_is_left(sum_swap(s)) = sum_is_right(s)
        }
        Sum.inr(y) {
            sum_swap_inr[T, U](y)
            sum_swap(s) = Sum.inl[U, T](y)
            sum_is_left_inl[U, T](y)
            sum_is_left(sum_swap(s)) = true
            sum_is_right_inr[T, U](y)
            sum_is_right(s) = true
            sum_is_left(sum_swap(s)) = sum_is_right(s)
        }
    }
}

/// Swapping interchanges the right-side predicate with the left-side predicate.
theorem sum_is_right_swap[T, U](s: Sum[T, U]) {
    sum_is_right(sum_swap(s)) = sum_is_left(s)
} by {
    match s {
        Sum.inl(x) {
            sum_swap_inl[T, U](x)
            sum_swap(s) = Sum.inr[U, T](x)
            sum_is_right_inr[U, T](x)
            sum_is_right(sum_swap(s)) = true
            sum_is_left_inl[T, U](x)
            sum_is_left(s) = true
            sum_is_right(sum_swap(s)) = sum_is_left(s)
        }
        Sum.inr(y) {
            sum_swap_inr[T, U](y)
            sum_swap(s) = Sum.inl[U, T](y)
            sum_is_right_inl[U, T](y)
            sum_is_right(sum_swap(s)) = false
            sum_is_left_inr[T, U](y)
            sum_is_left(s) = false
            sum_is_right(sum_swap(s)) = sum_is_left(s)
        }
    }
}

/// Swapping twice gives the original sum value.
theorem sum_swap_swap[T, U](s: Sum[T, U]) {
    sum_swap(sum_swap(s)) = s
} by {
    match s {
        Sum.inl(x) {
            sum_swap_inl[T, U](x)
            sum_swap(s) = Sum.inr[U, T](x)
            sum_swap_inr[U, T](x)
            sum_swap(sum_swap(s)) = Sum.inl[T, U](x)
            sum_swap(sum_swap(s)) = s
        }
        Sum.inr(y) {
            sum_swap_inr[T, U](y)
            sum_swap(s) = Sum.inl[U, T](y)
            sum_swap_inl[U, T](y)
            sum_swap(sum_swap(s)) = Sum.inr[T, U](y)
            sum_swap(sum_swap(s)) = s
        }
    }
}

/// The left-associated form of a triple sum.
define sum_assoc[T, U, V](s: Sum[Sum[T, U], V]) -> Sum[T, Sum[U, V]] {
    match s {
        Sum.inl(tu) {
            match tu {
                Sum.inl(t) {
                    Sum.inl[T, Sum[U, V]](t)
                }
                Sum.inr(u) {
                    Sum.inr[T, Sum[U, V]](Sum.inl[U, V](u))
                }
            }
        }
        Sum.inr(v) {
            Sum.inr[T, Sum[U, V]](Sum.inr[U, V](v))
        }
    }
}

/// The right-associated form of a triple sum.
define sum_unassoc[T, U, V](s: Sum[T, Sum[U, V]]) -> Sum[Sum[T, U], V] {
    match s {
        Sum.inl(t) {
            Sum.inl[Sum[T, U], V](Sum.inl[T, U](t))
        }
        Sum.inr(uv) {
            match uv {
                Sum.inl(u) {
                    Sum.inl[Sum[T, U], V](Sum.inr[T, U](u))
                }
                Sum.inr(v) {
                    Sum.inr[Sum[T, U], V](v)
                }
            }
        }
    }
}

/// Associating and then unassociating a triple sum gives the original sum value.
theorem sum_unassoc_assoc[T, U, V](s: Sum[Sum[T, U], V]) {
    sum_unassoc(sum_assoc(s)) = s
} by {
    match s {
        Sum.inl(tu) {
            match tu {
                Sum.inl(t) {
                    sum_assoc(s) = Sum.inl[T, Sum[U, V]](t)
                    sum_unassoc(sum_assoc(s)) = Sum.inl[Sum[T, U], V](Sum.inl[T, U](t))
                    sum_unassoc(sum_assoc(s)) = s
                }
                Sum.inr(u) {
                    sum_assoc(s) = Sum.inr[T, Sum[U, V]](Sum.inl[U, V](u))
                    sum_unassoc(sum_assoc(s)) = Sum.inl[Sum[T, U], V](Sum.inr[T, U](u))
                    sum_unassoc(sum_assoc(s)) = s
                }
            }
        }
        Sum.inr(v) {
            sum_assoc(s) = Sum.inr[T, Sum[U, V]](Sum.inr[U, V](v))
            sum_unassoc(sum_assoc(s)) = Sum.inr[Sum[T, U], V](v)
            sum_unassoc(sum_assoc(s)) = s
        }
    }
}

/// Unassociating and then associating a triple sum gives the original sum value.
theorem sum_assoc_unassoc[T, U, V](s: Sum[T, Sum[U, V]]) {
    sum_assoc(sum_unassoc(s)) = s
} by {
    match s {
        Sum.inl(t) {
            sum_unassoc(s) = Sum.inl[Sum[T, U], V](Sum.inl[T, U](t))
            sum_assoc(sum_unassoc(s)) = Sum.inl[T, Sum[U, V]](t)
            sum_assoc(sum_unassoc(s)) = s
        }
        Sum.inr(uv) {
            match uv {
                Sum.inl(u) {
                    sum_unassoc(s) = Sum.inl[Sum[T, U], V](Sum.inr[T, U](u))
                    sum_assoc(sum_unassoc(s)) = Sum.inr[T, Sum[U, V]](Sum.inl[U, V](u))
                    sum_assoc(sum_unassoc(s)) = s
                }
                Sum.inr(v) {
                    sum_unassoc(s) = Sum.inr[Sum[T, U], V](v)
                    sum_assoc(sum_unassoc(s)) = Sum.inr[T, Sum[U, V]](Sum.inr[U, V](v))
                    sum_assoc(sum_unassoc(s)) = s
                }
            }
        }
    }
}
