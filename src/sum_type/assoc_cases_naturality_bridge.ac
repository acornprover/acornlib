from sum_type.base import Sum, sum_cases, sum_map, sum_assoc, sum_unassoc,
    sum_cases_inl, sum_cases_inr, sum_map_inl, sum_map_inr

/// Nested case analysis for a left-associated triple sum.
define sum_cases_left_nested[T, U, V, W](
    s: Sum[Sum[T, U], V],
    left: T -> W,
    mid: U -> W,
    right: V -> W
) -> W {
    sum_cases(s, function(pair: Sum[T, U]) { sum_cases(pair, left, mid) }, right)
}

/// Nested case analysis for a right-associated triple sum.
define sum_cases_right_nested[T, U, V, W](
    s: Sum[T, Sum[U, V]],
    left: T -> W,
    mid: U -> W,
    right: V -> W
) -> W {
    sum_cases(s, left, function(rest: Sum[U, V]) { sum_cases(rest, mid, right) })
}

/// Mapping each component of a left-associated triple sum.
define sum_map_left_nested[A, B, C, D, E, F](
    s: Sum[Sum[A, B], C],
    f: A -> D,
    g: B -> E,
    h: C -> F
) -> Sum[Sum[D, E], F] {
    sum_map(s, function(pair: Sum[A, B]) { sum_map(pair, f, g) }, h)
}

/// Mapping each component of a right-associated triple sum.
define sum_map_right_nested[A, B, C, D, E, F](
    s: Sum[A, Sum[B, C]],
    f: A -> D,
    g: B -> E,
    h: C -> F
) -> Sum[D, Sum[E, F]] {
    sum_map(s, f, function(rest: Sum[B, C]) { sum_map(rest, g, h) })
}

/// Case analysis after associating a triple sum is nested case analysis on the left-associated input.
theorem sum_cases_assoc[T, U, V, W](
    s: Sum[Sum[T, U], V],
    left: T -> W,
    mid: U -> W,
    right: V -> W
) {
    sum_cases_right_nested(sum_assoc(s), left, mid, right) =
        sum_cases_left_nested(s, left, mid, right)
} by {
    match s {
        Sum.inl(pair) {
            match pair {
                Sum.inl(t) {
                    sum_assoc(s) = Sum.inl[T, Sum[U, V]](t)
                    sum_cases_inl[T, Sum[U, V], W](t, left,
                        function(rest: Sum[U, V]) { sum_cases(rest, mid, right) })
                    sum_cases_right_nested(sum_assoc(s), left, mid, right) = left(t)
                    sum_cases_inl[T, U, W](t, left, mid)
                    sum_cases(pair, left, mid) = left(t)
                    sum_cases_inl[Sum[T, U], V, W](pair,
                        function(ab: Sum[T, U]) { sum_cases(ab, left, mid) },
                        right)
                    sum_cases_left_nested(s, left, mid, right) = left(t)
                    sum_cases_right_nested(sum_assoc(s), left, mid, right) =
                        sum_cases_left_nested(s, left, mid, right)
                }
                Sum.inr(u) {
                    sum_assoc(s) = Sum.inr[T, Sum[U, V]](Sum.inl[U, V](u))
                    sum_cases_inl[U, V, W](u, mid, right)
                    sum_cases(Sum.inl[U, V](u), mid, right) = mid(u)
                    sum_cases_inr[T, Sum[U, V], W](Sum.inl[U, V](u), left,
                        function(rest: Sum[U, V]) { sum_cases(rest, mid, right) })
                    sum_cases_right_nested(sum_assoc(s), left, mid, right) = mid(u)
                    sum_cases_inr[T, U, W](u, left, mid)
                    sum_cases(pair, left, mid) = mid(u)
                    sum_cases_inl[Sum[T, U], V, W](pair,
                        function(ab: Sum[T, U]) { sum_cases(ab, left, mid) },
                        right)
                    sum_cases_left_nested(s, left, mid, right) = mid(u)
                    sum_cases_right_nested(sum_assoc(s), left, mid, right) =
                        sum_cases_left_nested(s, left, mid, right)
                }
            }
        }
        Sum.inr(v) {
            sum_assoc(s) = Sum.inr[T, Sum[U, V]](Sum.inr[U, V](v))
            sum_cases_inr[U, V, W](v, mid, right)
            sum_cases(Sum.inr[U, V](v), mid, right) = right(v)
            sum_cases_inr[T, Sum[U, V], W](Sum.inr[U, V](v), left,
                function(rest: Sum[U, V]) { sum_cases(rest, mid, right) })
            sum_cases_right_nested(sum_assoc(s), left, mid, right) = right(v)
            sum_cases_inr[Sum[T, U], V, W](v,
                function(ab: Sum[T, U]) { sum_cases(ab, left, mid) },
                right)
            sum_cases_left_nested(s, left, mid, right) = right(v)
            sum_cases_right_nested(sum_assoc(s), left, mid, right) =
                sum_cases_left_nested(s, left, mid, right)
        }
    }
}

/// Case analysis after unassociating a triple sum is nested case analysis on the right-associated input.
theorem sum_cases_unassoc[T, U, V, W](
    s: Sum[T, Sum[U, V]],
    left: T -> W,
    mid: U -> W,
    right: V -> W
) {
    sum_cases_left_nested(sum_unassoc(s), left, mid, right) =
        sum_cases_right_nested(s, left, mid, right)
} by {
    match s {
        Sum.inl(t) {
            sum_unassoc(s) = Sum.inl[Sum[T, U], V](Sum.inl[T, U](t))
            sum_cases_inl[T, U, W](t, left, mid)
            sum_cases(Sum.inl[T, U](t), left, mid) = left(t)
            sum_cases_inl[Sum[T, U], V, W](Sum.inl[T, U](t),
                function(ab: Sum[T, U]) { sum_cases(ab, left, mid) },
                right)
            sum_cases_left_nested(sum_unassoc(s), left, mid, right) = left(t)
            sum_cases_inl[T, Sum[U, V], W](t, left,
                function(rest: Sum[U, V]) { sum_cases(rest, mid, right) })
            sum_cases_right_nested(s, left, mid, right) = left(t)
            sum_cases_left_nested(sum_unassoc(s), left, mid, right) =
                sum_cases_right_nested(s, left, mid, right)
        }
        Sum.inr(rest) {
            match rest {
                Sum.inl(u) {
                    sum_unassoc(s) = Sum.inl[Sum[T, U], V](Sum.inr[T, U](u))
                    sum_cases_inr[T, U, W](u, left, mid)
                    sum_cases(Sum.inr[T, U](u), left, mid) = mid(u)
                    sum_cases_inl[Sum[T, U], V, W](Sum.inr[T, U](u),
                        function(ab: Sum[T, U]) { sum_cases(ab, left, mid) },
                        right)
                    sum_cases_left_nested(sum_unassoc(s), left, mid, right) = mid(u)
                    sum_cases_inl[U, V, W](u, mid, right)
                    sum_cases(rest, mid, right) = mid(u)
                    sum_cases_inr[T, Sum[U, V], W](rest, left,
                        function(uv: Sum[U, V]) { sum_cases(uv, mid, right) })
                    sum_cases_right_nested(s, left, mid, right) = mid(u)
                    sum_cases_left_nested(sum_unassoc(s), left, mid, right) =
                        sum_cases_right_nested(s, left, mid, right)
                }
                Sum.inr(v) {
                    sum_unassoc(s) = Sum.inr[Sum[T, U], V](v)
                    sum_cases_inr[Sum[T, U], V, W](v,
                        function(ab: Sum[T, U]) { sum_cases(ab, left, mid) },
                        right)
                    sum_cases_left_nested(sum_unassoc(s), left, mid, right) = right(v)
                    sum_cases_inr[U, V, W](v, mid, right)
                    sum_cases(rest, mid, right) = right(v)
                    sum_cases_inr[T, Sum[U, V], W](rest, left,
                        function(uv: Sum[U, V]) { sum_cases(uv, mid, right) })
                    sum_cases_right_nested(s, left, mid, right) = right(v)
                    sum_cases_left_nested(sum_unassoc(s), left, mid, right) =
                        sum_cases_right_nested(s, left, mid, right)
                }
            }
        }
    }
}

/// Associating commutes with mapping each component of a nested sum.
theorem sum_assoc_map_natural[A, B, C, D, E, F](
    f: A -> D,
    g: B -> E,
    h: C -> F,
    s: Sum[Sum[A, B], C]
) {
    sum_assoc(sum_map_left_nested(s, f, g, h)) =
        sum_map_right_nested(sum_assoc(s), f, g, h)
} by {
    match s {
        Sum.inl(pair) {
            match pair {
                Sum.inl(a) {
                    sum_map_inl[A, B, D, E](a, f, g)
                    sum_map(pair, f, g) = Sum.inl[D, E](f(a))
                    sum_map_inl[Sum[A, B], C, Sum[D, E], F](pair,
                        function(ab: Sum[A, B]) { sum_map(ab, f, g) }, h)
                    sum_map_left_nested(s, f, g, h) = Sum.inl[Sum[D, E], F](Sum.inl[D, E](f(a)))
                    sum_assoc(sum_map_left_nested(s, f, g, h)) = Sum.inl[D, Sum[E, F]](f(a))
                    sum_assoc(s) = Sum.inl[A, Sum[B, C]](a)
                    sum_map_inl[A, Sum[B, C], D, Sum[E, F]](a, f,
                        function(rest: Sum[B, C]) { sum_map(rest, g, h) })
                    sum_map_right_nested(sum_assoc(s), f, g, h) = Sum.inl[D, Sum[E, F]](f(a))
                    sum_assoc(sum_map_left_nested(s, f, g, h)) =
                        sum_map_right_nested(sum_assoc(s), f, g, h)
                }
                Sum.inr(b) {
                    sum_map_inr[A, B, D, E](b, f, g)
                    sum_map(pair, f, g) = Sum.inr[D, E](g(b))
                    sum_map_inl[Sum[A, B], C, Sum[D, E], F](pair,
                        function(ab: Sum[A, B]) { sum_map(ab, f, g) }, h)
                    sum_map_left_nested(s, f, g, h) = Sum.inl[Sum[D, E], F](Sum.inr[D, E](g(b)))
                    sum_assoc(sum_map_left_nested(s, f, g, h)) =
                        Sum.inr[D, Sum[E, F]](Sum.inl[E, F](g(b)))
                    sum_assoc(s) = Sum.inr[A, Sum[B, C]](Sum.inl[B, C](b))
                    sum_map_inl[B, C, E, F](b, g, h)
                    sum_map(Sum.inl[B, C](b), g, h) = Sum.inl[E, F](g(b))
                    sum_map_inr[A, Sum[B, C], D, Sum[E, F]](Sum.inl[B, C](b), f,
                        function(rest: Sum[B, C]) { sum_map(rest, g, h) })
                    sum_map_right_nested(sum_assoc(s), f, g, h) =
                        Sum.inr[D, Sum[E, F]](Sum.inl[E, F](g(b)))
                    sum_assoc(sum_map_left_nested(s, f, g, h)) =
                        sum_map_right_nested(sum_assoc(s), f, g, h)
                }
            }
        }
        Sum.inr(c) {
            sum_map_inr[Sum[A, B], C, Sum[D, E], F](c,
                function(ab: Sum[A, B]) { sum_map(ab, f, g) }, h)
            sum_map_left_nested(s, f, g, h) = Sum.inr[Sum[D, E], F](h(c))
            sum_assoc(sum_map_left_nested(s, f, g, h)) =
                Sum.inr[D, Sum[E, F]](Sum.inr[E, F](h(c)))
            sum_assoc(s) = Sum.inr[A, Sum[B, C]](Sum.inr[B, C](c))
            sum_map_inr[B, C, E, F](c, g, h)
            sum_map(Sum.inr[B, C](c), g, h) = Sum.inr[E, F](h(c))
            sum_map_inr[A, Sum[B, C], D, Sum[E, F]](Sum.inr[B, C](c), f,
                function(rest: Sum[B, C]) { sum_map(rest, g, h) })
            sum_map_right_nested(sum_assoc(s), f, g, h) =
                Sum.inr[D, Sum[E, F]](Sum.inr[E, F](h(c)))
            sum_assoc(sum_map_left_nested(s, f, g, h)) =
                sum_map_right_nested(sum_assoc(s), f, g, h)
        }
    }
}

/// Unassociating commutes with mapping each component of a nested sum.
theorem sum_unassoc_map_natural[A, B, C, D, E, F](
    f: A -> D,
    g: B -> E,
    h: C -> F,
    s: Sum[A, Sum[B, C]]
) {
    sum_unassoc(sum_map_right_nested(s, f, g, h)) =
        sum_map_left_nested(sum_unassoc(s), f, g, h)
} by {
    match s {
        Sum.inl(a) {
            sum_map_inl[A, Sum[B, C], D, Sum[E, F]](a, f,
                function(rest: Sum[B, C]) { sum_map(rest, g, h) })
            sum_map_right_nested(s, f, g, h) = Sum.inl[D, Sum[E, F]](f(a))
            sum_unassoc(sum_map_right_nested(s, f, g, h)) =
                Sum.inl[Sum[D, E], F](Sum.inl[D, E](f(a)))
            sum_unassoc(s) = Sum.inl[Sum[A, B], C](Sum.inl[A, B](a))
            sum_map_inl[A, B, D, E](a, f, g)
            sum_map(Sum.inl[A, B](a), f, g) = Sum.inl[D, E](f(a))
            sum_map_inl[Sum[A, B], C, Sum[D, E], F](Sum.inl[A, B](a),
                function(ab: Sum[A, B]) { sum_map(ab, f, g) }, h)
            sum_map_left_nested(sum_unassoc(s), f, g, h) =
                Sum.inl[Sum[D, E], F](Sum.inl[D, E](f(a)))
            sum_unassoc(sum_map_right_nested(s, f, g, h)) =
                sum_map_left_nested(sum_unassoc(s), f, g, h)
        }
        Sum.inr(rest) {
            match rest {
                Sum.inl(b) {
                    sum_map_inl[B, C, E, F](b, g, h)
                    sum_map(rest, g, h) = Sum.inl[E, F](g(b))
                    sum_map_inr[A, Sum[B, C], D, Sum[E, F]](rest, f,
                        function(uv: Sum[B, C]) { sum_map(uv, g, h) })
                    sum_map_right_nested(s, f, g, h) =
                        Sum.inr[D, Sum[E, F]](Sum.inl[E, F](g(b)))
                    sum_unassoc(sum_map_right_nested(s, f, g, h)) =
                        Sum.inl[Sum[D, E], F](Sum.inr[D, E](g(b)))
                    sum_unassoc(s) = Sum.inl[Sum[A, B], C](Sum.inr[A, B](b))
                    sum_map_inr[A, B, D, E](b, f, g)
                    sum_map(Sum.inr[A, B](b), f, g) = Sum.inr[D, E](g(b))
                    sum_map_inl[Sum[A, B], C, Sum[D, E], F](Sum.inr[A, B](b),
                        function(ab: Sum[A, B]) { sum_map(ab, f, g) }, h)
                    sum_map_left_nested(sum_unassoc(s), f, g, h) =
                        Sum.inl[Sum[D, E], F](Sum.inr[D, E](g(b)))
                    sum_unassoc(sum_map_right_nested(s, f, g, h)) =
                        sum_map_left_nested(sum_unassoc(s), f, g, h)
                }
                Sum.inr(c) {
                    sum_map_inr[B, C, E, F](c, g, h)
                    sum_map(rest, g, h) = Sum.inr[E, F](h(c))
                    sum_map_inr[A, Sum[B, C], D, Sum[E, F]](rest, f,
                        function(uv: Sum[B, C]) { sum_map(uv, g, h) })
                    sum_map_right_nested(s, f, g, h) =
                        Sum.inr[D, Sum[E, F]](Sum.inr[E, F](h(c)))
                    sum_unassoc(sum_map_right_nested(s, f, g, h)) =
                        Sum.inr[Sum[D, E], F](h(c))
                    sum_unassoc(s) = Sum.inr[Sum[A, B], C](c)
                    sum_map_inr[Sum[A, B], C, Sum[D, E], F](c,
                        function(ab: Sum[A, B]) { sum_map(ab, f, g) }, h)
                    sum_map_left_nested(sum_unassoc(s), f, g, h) =
                        Sum.inr[Sum[D, E], F](h(c))
                    sum_unassoc(sum_map_right_nested(s, f, g, h)) =
                        sum_map_left_nested(sum_unassoc(s), f, g, h)
                }
            }
        }
    }
}
