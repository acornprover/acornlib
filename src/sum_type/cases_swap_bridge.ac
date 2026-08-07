from data.basic.functions import compose
from sum_type.base import Sum, sum_cases, sum_map_left, sum_map_right, sum_swap,
    sum_cases_inl, sum_cases_inr,
    sum_map_left_inl, sum_map_left_inr,
    sum_map_right_inl, sum_map_right_inr,
    sum_swap_inl, sum_swap_inr

/// Case analysis after swapping a sum interchanges the two branches.
theorem sum_cases_swap[T, U, V](s: Sum[T, U], left: U -> V, right: T -> V) {
    sum_cases(sum_swap(s), left, right) = sum_cases(s, right, left)
} by {
    match s {
        Sum.inl(x) {
            sum_swap_inl[T, U](x)
            sum_swap(s) = Sum.inr[U, T](x)
            sum_cases_inr[U, T, V](x, left, right)
            sum_cases(sum_swap(s), left, right) = right(x)
            sum_cases_inl[T, U, V](x, right, left)
            sum_cases(s, right, left) = right(x)
            sum_cases(sum_swap(s), left, right) = sum_cases(s, right, left)
        }
        Sum.inr(y) {
            sum_swap_inr[T, U](y)
            sum_swap(s) = Sum.inl[U, T](y)
            sum_cases_inl[U, T, V](y, left, right)
            sum_cases(sum_swap(s), left, right) = left(y)
            sum_cases_inr[T, U, V](y, right, left)
            sum_cases(s, right, left) = left(y)
            sum_cases(sum_swap(s), left, right) = sum_cases(s, right, left)
        }
    }
}

/// Case analysis after mapping the left side composes the left branch with the map.
theorem sum_cases_map_left[T, U, V, W](s: Sum[T, U], f: T -> V, left: V -> W, right: U -> W) {
    sum_cases(sum_map_left(s, f), left, right) = sum_cases(s, compose(left, f), right)
} by {
    match s {
        Sum.inl(x) {
            sum_map_left_inl[T, U, V](x, f)
            sum_map_left(s, f) = Sum.inl[V, U](f(x))
            sum_cases_inl[V, U, W](f(x), left, right)
            sum_cases(sum_map_left(s, f), left, right) = left(f(x))
            sum_cases_inl[T, U, W](x, compose(left, f), right)
            compose(left, f, x) = left(f(x))
            sum_cases(s, compose(left, f), right) = left(f(x))
            sum_cases(sum_map_left(s, f), left, right) = sum_cases(s, compose(left, f), right)
        }
        Sum.inr(y) {
            sum_map_left_inr[T, U, V](y, f)
            sum_map_left(s, f) = Sum.inr[V, U](y)
            sum_cases_inr[V, U, W](y, left, right)
            sum_cases(sum_map_left(s, f), left, right) = right(y)
            sum_cases_inr[T, U, W](y, compose(left, f), right)
            sum_cases(s, compose(left, f), right) = right(y)
            sum_cases(sum_map_left(s, f), left, right) = sum_cases(s, compose(left, f), right)
        }
    }
}

/// Case analysis after mapping the right side composes the right branch with the map.
theorem sum_cases_map_right[T, U, V, W](s: Sum[T, U], g: U -> V, left: T -> W, right: V -> W) {
    sum_cases(sum_map_right(s, g), left, right) = sum_cases(s, left, compose(right, g))
} by {
    match s {
        Sum.inl(x) {
            sum_map_right_inl[T, U, V](x, g)
            sum_map_right(s, g) = Sum.inl[T, V](x)
            sum_cases_inl[T, V, W](x, left, right)
            sum_cases(sum_map_right(s, g), left, right) = left(x)
            sum_cases_inl[T, U, W](x, left, compose(right, g))
            sum_cases(s, left, compose(right, g)) = left(x)
            sum_cases(sum_map_right(s, g), left, right) = sum_cases(s, left, compose(right, g))
        }
        Sum.inr(y) {
            sum_map_right_inr[T, U, V](y, g)
            sum_map_right(s, g) = Sum.inr[T, V](g(y))
            sum_cases_inr[T, V, W](g(y), left, right)
            sum_cases(sum_map_right(s, g), left, right) = right(g(y))
            sum_cases_inr[T, U, W](y, left, compose(right, g))
            compose(right, g, y) = right(g(y))
            sum_cases(s, left, compose(right, g)) = right(g(y))
            sum_cases(sum_map_right(s, g), left, right) = sum_cases(s, left, compose(right, g))
        }
    }
}

/// Mapping the left side and then the right side commutes with doing the maps in the opposite order.
theorem sum_map_left_right_commute[T, U, V, W](s: Sum[T, U], f: T -> V, g: U -> W) {
    sum_map_right(sum_map_left(s, f), g) = sum_map_left(sum_map_right(s, g), f)
} by {
    match s {
        Sum.inl(x) {
            sum_map_left_inl[T, U, V](x, f)
            sum_map_left(s, f) = Sum.inl[V, U](f(x))
            sum_map_right_inl[V, U, W](f(x), g)
            sum_map_right(sum_map_left(s, f), g) = Sum.inl[V, W](f(x))
            sum_map_right_inl[T, U, W](x, g)
            sum_map_right(s, g) = Sum.inl[T, W](x)
            sum_map_left_inl[T, W, V](x, f)
            sum_map_left(sum_map_right(s, g), f) = Sum.inl[V, W](f(x))
            sum_map_right(sum_map_left(s, f), g) = sum_map_left(sum_map_right(s, g), f)
        }
        Sum.inr(y) {
            sum_map_left_inr[T, U, V](y, f)
            sum_map_left(s, f) = Sum.inr[V, U](y)
            sum_map_right_inr[V, U, W](y, g)
            sum_map_right(sum_map_left(s, f), g) = Sum.inr[V, W](g(y))
            sum_map_right_inr[T, U, W](y, g)
            sum_map_right(s, g) = Sum.inr[T, W](g(y))
            sum_map_left_inr[T, W, V](g(y), f)
            sum_map_left(sum_map_right(s, g), f) = Sum.inr[V, W](g(y))
            sum_map_right(sum_map_left(s, f), g) = sum_map_left(sum_map_right(s, g), f)
        }
    }
}

/// Swapping after mapping the left side is mapping the right side after swapping.
theorem sum_swap_map_left[T, U, V](s: Sum[T, U], f: T -> V) {
    sum_swap(sum_map_left(s, f)) = sum_map_right(sum_swap(s), f)
} by {
    match s {
        Sum.inl(x) {
            sum_map_left_inl[T, U, V](x, f)
            sum_map_left(s, f) = Sum.inl[V, U](f(x))
            sum_swap_inl[V, U](f(x))
            sum_swap(sum_map_left(s, f)) = Sum.inr[U, V](f(x))
            sum_swap_inl[T, U](x)
            sum_swap(s) = Sum.inr[U, T](x)
            sum_map_right_inr[U, T, V](x, f)
            sum_map_right(sum_swap(s), f) = Sum.inr[U, V](f(x))
            sum_swap(sum_map_left(s, f)) = sum_map_right(sum_swap(s), f)
        }
        Sum.inr(y) {
            sum_map_left_inr[T, U, V](y, f)
            sum_map_left(s, f) = Sum.inr[V, U](y)
            sum_swap_inr[V, U](y)
            sum_swap(sum_map_left(s, f)) = Sum.inl[U, V](y)
            sum_swap_inr[T, U](y)
            sum_swap(s) = Sum.inl[U, T](y)
            sum_map_right_inl[U, T, V](y, f)
            sum_map_right(sum_swap(s), f) = Sum.inl[U, V](y)
            sum_swap(sum_map_left(s, f)) = sum_map_right(sum_swap(s), f)
        }
    }
}

/// Swapping after mapping the right side is mapping the left side after swapping.
theorem sum_swap_map_right[T, U, V](s: Sum[T, U], g: U -> V) {
    sum_swap(sum_map_right(s, g)) = sum_map_left(sum_swap(s), g)
} by {
    match s {
        Sum.inl(x) {
            sum_map_right_inl[T, U, V](x, g)
            sum_map_right(s, g) = Sum.inl[T, V](x)
            sum_swap_inl[T, V](x)
            sum_swap(sum_map_right(s, g)) = Sum.inr[V, T](x)
            sum_swap_inl[T, U](x)
            sum_swap(s) = Sum.inr[U, T](x)
            sum_map_left_inr[U, T, V](x, g)
            sum_map_left(sum_swap(s), g) = Sum.inr[V, T](x)
            sum_swap(sum_map_right(s, g)) = sum_map_left(sum_swap(s), g)
        }
        Sum.inr(y) {
            sum_map_right_inr[T, U, V](y, g)
            sum_map_right(s, g) = Sum.inr[T, V](g(y))
            sum_swap_inr[T, V](g(y))
            sum_swap(sum_map_right(s, g)) = Sum.inl[V, T](g(y))
            sum_swap_inr[T, U](y)
            sum_swap(s) = Sum.inl[U, T](y)
            sum_map_left_inl[U, T, V](y, g)
            sum_map_left(sum_swap(s), g) = Sum.inl[V, T](g(y))
            sum_swap(sum_map_right(s, g)) = sum_map_left(sum_swap(s), g)
        }
    }
}
