from data.basic.functions import is_injective_fn, is_surjective_fn, is_bijection_fn,
    injective_fn_eq, surjective_fn_has_preimage,
    bijection_fn_is_injective, bijection_fn_is_surjective
from sum_type.base import Sum, sum_map, sum_map_inl, sum_map_inr

/// The curried sum map as a function between sum types.
define sum_map_fn[A, B, C, D](f: A -> C, g: B -> D, s: Sum[A, B]) -> Sum[C, D] {
    sum_map(s, f, g)
}

/// If both side maps are injective, then the induced map on sums is injective.
theorem sum_map_injective_fn[A, B, C, D](f: A -> C, g: B -> D) {
    is_injective_fn(f) and is_injective_fn(g) implies is_injective_fn(sum_map_fn(f, g))
} by {
    if is_injective_fn(f) and is_injective_fn(g) {
        forall(s1: Sum[A, B], s2: Sum[A, B]) {
            if sum_map_fn(f, g, s1) = sum_map_fn(f, g, s2) {
                match s1 {
                    Sum.inl(a1) {
                        sum_map_inl[A, B, C, D](a1, f, g)
                        sum_map_fn(f, g, s1) = Sum.inl[C, D](f(a1))
                        match s2 {
                            Sum.inl(a2) {
                                sum_map_inl[A, B, C, D](a2, f, g)
                                sum_map_fn(f, g, s2) = Sum.inl[C, D](f(a2))
                                Sum.inl[C, D](f(a1)) = Sum.inl[C, D](f(a2))
                                f(a1) = f(a2)
                                injective_fn_eq[A, C](f, a1, a2)
                                a1 = a2
                                s1 = s2
                            }
                            Sum.inr(b2) {
                                sum_map_inr[A, B, C, D](b2, f, g)
                                sum_map_fn(f, g, s2) = Sum.inr[C, D](g(b2))
                                Sum.inl[C, D](f(a1)) = Sum.inr[C, D](g(b2))
                                false
                            }
                        }
                    }
                    Sum.inr(b1) {
                        sum_map_inr[A, B, C, D](b1, f, g)
                        sum_map_fn(f, g, s1) = Sum.inr[C, D](g(b1))
                        match s2 {
                            Sum.inl(a2) {
                                sum_map_inl[A, B, C, D](a2, f, g)
                                sum_map_fn(f, g, s2) = Sum.inl[C, D](f(a2))
                                Sum.inr[C, D](g(b1)) = Sum.inl[C, D](f(a2))
                                false
                            }
                            Sum.inr(b2) {
                                sum_map_inr[A, B, C, D](b2, f, g)
                                sum_map_fn(f, g, s2) = Sum.inr[C, D](g(b2))
                                Sum.inr[C, D](g(b1)) = Sum.inr[C, D](g(b2))
                                g(b1) = g(b2)
                                injective_fn_eq[B, D](g, b1, b2)
                                b1 = b2
                                s1 = s2
                            }
                        }
                    }
                }
            }
        }
    }
}

/// If both side maps are surjective, then the induced map on sums is surjective.
theorem sum_map_surjective_fn[A, B, C, D](f: A -> C, g: B -> D) {
    is_surjective_fn(f) and is_surjective_fn(g) implies is_surjective_fn(sum_map_fn(f, g))
} by {
    if is_surjective_fn(f) and is_surjective_fn(g) {
        forall(target: Sum[C, D]) {
            match target {
                Sum.inl(c) {
                    surjective_fn_has_preimage[A, C](f, c)
                    let a: A satisfy {
                        f(a) = c
                    }
                    let source: Sum[A, B] = Sum.inl[A, B](a)
                    sum_map_inl[A, B, C, D](a, f, g)
                    sum_map_fn(f, g, source) = Sum.inl[C, D](f(a))
                    sum_map_fn(f, g, source) = target
                    exists(source0: Sum[A, B]) {
                        sum_map_fn(f, g, source0) = target
                    }
                }
                Sum.inr(d) {
                    surjective_fn_has_preimage[B, D](g, d)
                    let b: B satisfy {
                        g(b) = d
                    }
                    let source: Sum[A, B] = Sum.inr[A, B](b)
                    sum_map_inr[A, B, C, D](b, f, g)
                    sum_map_fn(f, g, source) = Sum.inr[C, D](g(b))
                    sum_map_fn(f, g, source) = target
                    exists(source0: Sum[A, B]) {
                        sum_map_fn(f, g, source0) = target
                    }
                }
            }
        }
    }
}

/// If both side maps are bijections, then the induced map on sums is a bijection.
theorem sum_map_bijection_fn[A, B, C, D](f: A -> C, g: B -> D) {
    is_bijection_fn(f) and is_bijection_fn(g) implies is_bijection_fn(sum_map_fn(f, g))
} by {
    if is_bijection_fn(f) and is_bijection_fn(g) {
        bijection_fn_is_injective[A, C](f)
        bijection_fn_is_injective[B, D](g)
        is_injective_fn(f)
        is_injective_fn(g)
        sum_map_injective_fn[A, B, C, D](f, g)
        is_injective_fn(sum_map_fn(f, g))
        bijection_fn_is_surjective[A, C](f)
        bijection_fn_is_surjective[B, D](g)
        is_surjective_fn(f)
        is_surjective_fn(g)
        sum_map_surjective_fn[A, B, C, D](f, g)
        is_surjective_fn(sum_map_fn(f, g))
        is_bijection_fn(sum_map_fn(f, g))
    }
}
