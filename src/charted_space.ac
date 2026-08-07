/// Charted spaces: a space equipped with an atlas of charts into a model space.

from chart import Chart, chart_left_inv, chart_right_inv, chart_map_source, chart_map_target,
    chart_to_fun_inj_on_source, chart_inv_fun_inj_on_target
from data.basic.set import Set

/// True if `atlas` is a set of charts and `chart_at` selects, for each point,
/// a chart in the atlas whose source contains that point.
define is_charted_space_data[M, E](atlas: Set[Chart[M, E]],
    chart_at: M -> Chart[M, E]) -> Bool {
    forall(x: M) {
        chart_at(x).source.contains(x)
    } and forall(x: M) {
        atlas.contains(chart_at(x))
    }
}

/// A charted space on `M` modeled on `E`, equipped with an atlas of charts
/// and a chosen chart around each point.
structure ChartedSpace[M, E] {
    /// The atlas of charts on the space.
    atlas: Set[Chart[M, E]]

    /// For each point, a chosen chart in the atlas whose source contains it.
    chart_at: M -> Chart[M, E]
} constraint {
    is_charted_space_data(atlas, chart_at)
}

/// Every point lies in the source of the chart chosen at that point.
theorem charted_space_mem_chart_source[M, E](c: ChartedSpace[M, E], x: M) {
    c.chart_at(x).source.contains(x)
} by {
    ChartedSpace[M, E].constraint(c.atlas, c.chart_at)
    is_charted_space_data(c.atlas, c.chart_at) = (forall(a: M) {
        c.chart_at(a).source.contains(a)
    } and forall(a: M) {
        c.atlas.contains(c.chart_at(a))
    })
}

/// The chart chosen at each point belongs to the atlas.
theorem charted_space_chart_mem_atlas[M, E](c: ChartedSpace[M, E], x: M) {
    c.atlas.contains(c.chart_at(x))
} by {
    is_charted_space_data(c.atlas, c.chart_at) = (forall(a: M) {
        c.chart_at(a).source.contains(a)
    } and forall(a: M) {
        c.atlas.contains(c.chart_at(a))
    })
}

/// Every point of a charted space lies in the source of some chart from the atlas.
theorem charted_space_atlas_covers[M, E](c: ChartedSpace[M, E], x: M) {
    exists(ch: Chart[M, E]) {
        c.atlas.contains(ch) and ch.source.contains(x)
    }
} by {
    charted_space_mem_chart_source(c, x)
    charted_space_chart_mem_atlas(c, x)
}

/// The chosen chart at a point maps that point into its target.
theorem charted_space_chart_at_map_source[M, E](c: ChartedSpace[M, E], x: M) {
    c.chart_at(x).target.contains(c.chart_at(x).to_fun(x))
} by {
    charted_space_mem_chart_source(c, x)
    chart_map_source(c.chart_at(x), x)
}

/// The inverse coordinate map of the chosen chart at a point recovers that point.
theorem charted_space_chart_at_left_inv[M, E](c: ChartedSpace[M, E], x: M) {
    c.chart_at(x).inv_fun(c.chart_at(x).to_fun(x)) = x
} by {
    charted_space_mem_chart_source(c, x)
    chart_left_inv(c.chart_at(x), x)
}

/// The inverse coordinate map of a chosen chart sends target coordinates back into its source.
theorem charted_space_chart_at_map_target[M, E](c: ChartedSpace[M, E], p: M, y: E) {
    c.chart_at(p).target.contains(y) implies c.chart_at(p).source.contains(c.chart_at(p).inv_fun(y))
} by {
    chart_map_target(c.chart_at(p), y)
}

/// On target coordinates of a chosen chart, forward coordinates undo the inverse map.
theorem charted_space_chart_at_right_inv[M, E](c: ChartedSpace[M, E], p: M, y: E) {
    c.chart_at(p).target.contains(y) implies c.chart_at(p).to_fun(c.chart_at(p).inv_fun(y)) = y
} by {
    chart_right_inv(c.chart_at(p), y)
}

/// Every point has an atlas chart whose coordinate image is in target and round-trips back.
theorem charted_space_atlas_covers_with_coordinate[M, E](c: ChartedSpace[M, E], x: M) {
    exists(ch: Chart[M, E]) {
        c.atlas.contains(ch) and ch.source.contains(x) and ch.target.contains(ch.to_fun(x)) and
            ch.inv_fun(ch.to_fun(x)) = x
    }
} by {
    charted_space_mem_chart_source(c, x)
    charted_space_chart_mem_atlas(c, x)
    chart_map_source(c.chart_at(x), x)
    chart_left_inv(c.chart_at(x), x)
}

/// The forward coordinate map of the chosen chart at a point is injective on its source.
theorem charted_space_chart_at_to_fun_inj[M, E](c: ChartedSpace[M, E], p: M, x: M, y: M) {
    c.chart_at(p).source.contains(x) and c.chart_at(p).source.contains(y)
        and c.chart_at(p).to_fun(x) = c.chart_at(p).to_fun(y)
        implies x = y
} by {
    chart_to_fun_inj_on_source(c.chart_at(p), x, y)
}

/// The inverse coordinate map of the chosen chart at a point is injective on its target.
theorem charted_space_chart_at_inv_fun_inj[M, E](c: ChartedSpace[M, E], p: M, a: E, b: E) {
    c.chart_at(p).target.contains(a) and c.chart_at(p).target.contains(b)
        and c.chart_at(p).inv_fun(a) = c.chart_at(p).inv_fun(b)
        implies a = b
} by {
    chart_inv_fun_inj_on_target(c.chart_at(p), a, b)
}

/// A chart whose source is the whole space yields a charted space with a
/// single-chart atlas, picking that chart at every point.
theorem charted_space_of_global_chart_constructible[M, E](c: Chart[M, E]) {
    c.source = Set[M].universal_set implies exists(cs: ChartedSpace[M, E]) {
        ChartedSpace[M, E].new(Set[Chart[M, E]].singleton(c),
            constant[M, Chart[M, E]](c)) = Option.some(cs)
    }
} by {
    if c.source = Set[M].universal_set {
        forall(x: M) {
            c.source.contains(x)
            constant[M, Chart[M, E]](c)(x).source.contains(x)
        }
        let p1: Bool = forall(x: M) {
            constant[M, Chart[M, E]](c)(x).source.contains(x)
        }
        p1
        forall(x: M) {
            Set[Chart[M, E]].singleton(c).contains(constant[M, Chart[M, E]](c)(x))
        }
        let p2: Bool = forall(x: M) {
            Set[Chart[M, E]].singleton(c).contains(constant[M, Chart[M, E]](c)(x))
        }
        p2
        is_charted_space_data(Set[Chart[M, E]].singleton(c),
            constant[M, Chart[M, E]](c)) = (forall(x: M) {
            constant[M, Chart[M, E]](c)(x).source.contains(x)
        } and forall(x: M) {
            Set[Chart[M, E]].singleton(c).contains(constant[M, Chart[M, E]](c)(x))
        })
        is_charted_space_data(Set[Chart[M, E]].singleton(c),
            constant[M, Chart[M, E]](c))
    }
}
