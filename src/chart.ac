/// Charts on a space, modeled as partial bijections into a model space.

from data.basic.functions import compose, identity_fn
from analysis import LocalEquiv, is_local_equiv_data, local_equiv_data_id,
    local_equiv_data_left_inv, local_equiv_data_right_inv, local_equiv_data_map_source,
    local_equiv_data_map_target, local_equiv_data_restr, local_equiv_data_swap,
    local_equiv_data_trans, local_equiv_restr_constructible,
    local_equiv_swap_constructible
from pair import curry, uncurry
from data.basic.set import Set, set_preimage

/// A chart on `M` with values in the model space `E`,
/// modeled as a partial bijection between `M` and `E`.
structure Chart[M, E] {
    /// The underlying partial bijection from the chart domain into the model space.
    to_local_equiv: LocalEquiv[M, E]
}

attributes Chart[M, E] {
    /// The set on which the chart is defined.
    define source(self) -> Set[M] {
        self.to_local_equiv.source
    }

    /// The image of the chart inside the model space.
    define target(self) -> Set[E] {
        self.to_local_equiv.target
    }

    /// The forward coordinate map of the chart.
    define to_fun(self) -> (M -> E) {
        self.to_local_equiv.to_fun
    }

    /// The inverse coordinate map of the chart.
    define inv_fun(self) -> (E -> M) {
        self.to_local_equiv.inv_fun
    }
}

/// On the source, the inverse coordinate map undoes the forward coordinate map.
theorem chart_left_inv[M, E](c: Chart[M, E], x: M) {
    c.source.contains(x) implies c.inv_fun(c.to_fun(x)) = x
} by {
    if c.source.contains(x) {
        LocalEquiv[M, E].constraint(c.to_local_equiv.source, c.to_local_equiv.target,
            c.to_local_equiv.to_fun, c.to_local_equiv.inv_fun)
        local_equiv_data_left_inv(c.to_local_equiv.source, c.to_local_equiv.target,
            c.to_local_equiv.to_fun, c.to_local_equiv.inv_fun, x)
        c.to_local_equiv.inv_fun(c.to_local_equiv.to_fun(x)) = x
    }
}

/// On the target, the forward coordinate map undoes the inverse coordinate map.
theorem chart_right_inv[M, E](c: Chart[M, E], y: E) {
    c.target.contains(y) implies c.to_fun(c.inv_fun(y)) = y
} by {
    if c.target.contains(y) {
        LocalEquiv[M, E].constraint(c.to_local_equiv.source, c.to_local_equiv.target,
            c.to_local_equiv.to_fun, c.to_local_equiv.inv_fun)
        local_equiv_data_right_inv(c.to_local_equiv.source, c.to_local_equiv.target,
            c.to_local_equiv.to_fun, c.to_local_equiv.inv_fun, y)
        c.to_local_equiv.to_fun(c.to_local_equiv.inv_fun(y)) = y
    }
}

/// The forward coordinate map sends the source of a chart into its target.
theorem chart_map_source[M, E](c: Chart[M, E], x: M) {
    c.source.contains(x) implies c.target.contains(c.to_fun(x))
} by {
    if c.source.contains(x) {
        local_equiv_data_map_source(c.to_local_equiv.source, c.to_local_equiv.target,
            c.to_local_equiv.to_fun, c.to_local_equiv.inv_fun, x)
        c.to_local_equiv.target.contains(c.to_local_equiv.to_fun(x))
    }
}

/// The inverse coordinate map sends the target of a chart into its source.
theorem chart_map_target[M, E](c: Chart[M, E], y: E) {
    c.target.contains(y) implies c.source.contains(c.inv_fun(y))
} by {
    if c.target.contains(y) {
        local_equiv_data_map_target(c.to_local_equiv.source, c.to_local_equiv.target,
            c.to_local_equiv.to_fun, c.to_local_equiv.inv_fun, y)
        c.to_local_equiv.source.contains(c.to_local_equiv.inv_fun(y))
    }
}

/// The forward coordinate map of a chart is injective on the source.
theorem chart_to_fun_inj_on_source[M, E](c: Chart[M, E], x: M, y: M) {
    c.source.contains(x) and c.source.contains(y) and c.to_fun(x) = c.to_fun(y)
        implies x = y
} by {
    if c.source.contains(x) and c.source.contains(y) and c.to_fun(x) = c.to_fun(y) {
        chart_left_inv(c, x)
        chart_left_inv(c, y)
        x = y
    }
}

/// The inverse coordinate map of a chart is injective on the target.
theorem chart_inv_fun_inj_on_target[M, E](c: Chart[M, E], a: E, b: E) {
    c.target.contains(a) and c.target.contains(b) and c.inv_fun(a) = c.inv_fun(b)
        implies a = b
} by {
    if c.target.contains(a) and c.target.contains(b) and c.inv_fun(a) = c.inv_fun(b) {
        chart_right_inv(c, a)
        chart_right_inv(c, b)
        a = b
    }
}

/// The identity chart on a subset of a space, using the identity as both coordinate maps.
theorem chart_id_constructible[M](s: Set[M]) {
    exists(e: LocalEquiv[M, M]) {
        LocalEquiv[M, M].new(s, s, identity_fn[M], identity_fn[M]) = Option.some(e)
    }
} by {
    local_equiv_data_id(s)
}

/// Restricting a chart to a subset of its source yields a partial bijection
/// on the same model space, with forward map `c.to_fun` on `c.source ∩ s` and
/// inverse `c.inv_fun` on `c.target ∩ c.inv_fun⁻¹(s)`.
theorem chart_restr_constructible[M, E](c: Chart[M, E], s: Set[M]) {
    exists(le: LocalEquiv[M, E]) {
        LocalEquiv[M, E].new(c.source.intersection(s),
            c.target.intersection(set_preimage(c.inv_fun, s)),
            c.to_fun, c.inv_fun) = Option.some(le)
    }
} by {
    local_equiv_data_restr(c.to_local_equiv.source, c.to_local_equiv.target,
        c.to_local_equiv.to_fun, c.to_local_equiv.inv_fun, s)
    is_local_equiv_data(c.to_local_equiv.source.intersection(s),
        c.to_local_equiv.target.intersection(set_preimage(c.to_local_equiv.inv_fun, s)),
        c.to_local_equiv.to_fun, c.to_local_equiv.inv_fun)
    is_local_equiv_data(c.source.intersection(s),
        c.target.intersection(set_preimage(c.inv_fun, s)), c.to_fun, c.inv_fun)
}

/// The inverse of a chart, viewed as a partial bijection from the model space
/// back to `M`, swapping source/target and forward/inverse coordinate maps.
theorem chart_symm_constructible[M, E](c: Chart[M, E]) {
    exists(le: LocalEquiv[E, M]) {
        LocalEquiv[E, M].new(c.target, c.source, c.inv_fun, c.to_fun) = Option.some(le)
    }
} by {
    local_equiv_data_swap(c.to_local_equiv.source, c.to_local_equiv.target,
        c.to_local_equiv.to_fun, c.to_local_equiv.inv_fun)
    is_local_equiv_data(c.to_local_equiv.target, c.to_local_equiv.source,
        c.to_local_equiv.inv_fun, c.to_local_equiv.to_fun)
    is_local_equiv_data(c.target, c.source, c.inv_fun, c.to_fun)
}

/// The transition map from one chart to another, viewed as a partial bijection on
/// the model space, is constructible. It goes from `c1.target` (restricted to the
/// preimage under `c1.inv_fun` of `c2.source`) to `c2.target` (restricted to the
/// preimage under `c2.inv_fun` of `c1.source`), with forward map `c2.to_fun ∘ c1.inv_fun`
/// and inverse `c1.to_fun ∘ c2.inv_fun`.
theorem chart_transition_constructible[M, E](c1: Chart[M, E], c2: Chart[M, E]) {
    exists(e: LocalEquiv[E, E]) {
        LocalEquiv[E, E].new(
            c1.target.intersection(set_preimage(c1.inv_fun, c2.source)),
            c2.target.intersection(set_preimage(c2.inv_fun, c1.source)),
            compose(c2.to_fun, c1.inv_fun),
            compose(c1.to_fun, c2.inv_fun)) = Option.some(e)
    }
} by {
    LocalEquiv[M, E].constraint(c1.to_local_equiv.source, c1.to_local_equiv.target,
        c1.to_local_equiv.to_fun, c1.to_local_equiv.inv_fun)
    LocalEquiv[M, E].constraint(c2.to_local_equiv.source, c2.to_local_equiv.target,
        c2.to_local_equiv.to_fun, c2.to_local_equiv.inv_fun)
    local_equiv_data_swap(c1.to_local_equiv.source, c1.to_local_equiv.target,
        c1.to_local_equiv.to_fun, c1.to_local_equiv.inv_fun)
    local_equiv_data_trans(
        c1.to_local_equiv.target, c1.to_local_equiv.source,
        c1.to_local_equiv.inv_fun, c1.to_local_equiv.to_fun,
        c2.to_local_equiv.source, c2.to_local_equiv.target,
        c2.to_local_equiv.to_fun, c2.to_local_equiv.inv_fun)
    is_local_equiv_data(
        c1.to_local_equiv.target.intersection(set_preimage(c1.to_local_equiv.inv_fun, c2.to_local_equiv.source)),
        c2.to_local_equiv.target.intersection(set_preimage(c2.to_local_equiv.inv_fun, c1.to_local_equiv.source)),
        compose(c2.to_local_equiv.to_fun, c1.to_local_equiv.inv_fun),
        compose(c1.to_local_equiv.to_fun, c2.to_local_equiv.inv_fun))
    is_local_equiv_data(
        c1.target.intersection(set_preimage(c1.inv_fun, c2.source)),
        c2.target.intersection(set_preimage(c2.inv_fun, c1.source)),
        compose(c2.to_fun, c1.inv_fun),
        compose(c1.to_fun, c2.inv_fun))
}
