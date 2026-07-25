from order import LinearOrder, lte_refl, gte_refl, lte_trans, gte_trans, lte_antisymm, lt_imp_lte, gt_imp_gte, lt_imp_ne, gt_imp_ne, not_lte_imp_gt, not_gte_imp_lt, not_lt_imp_gte, not_gt_imp_lte, eq_or_lt_of_lte, eq_or_gt_of_gte, lte_or_gte, lte_or_gt, gte_or_lt, min_symm, max_symm, min_lte_left, min_lte_right, lte_max_left, lte_max_right, min_eq_left_of_lte, min_eq_right_of_gte, max_eq_left_of_gte, max_eq_right_of_lte, min_eq_left_iff_lte, min_eq_right_iff_gte, max_eq_right_iff_lte, max_eq_left_iff_gte, min_imp_lte, max_imp_gte, lt_of_lt_of_lte, lt_of_lte_of_lt, min_is_one, max_is_one, min_idem, max_idem, gt_of_ne_of_gte, gt_or_lte, lt_max_iff, lt_max_of_left_or_right, lt_min_iff, lt_min_imp, lt_min_of_bounds, lt_of_ne_of_lte, lt_or_gte, lte_max_iff, lte_max_imp, lte_max_of_left, lte_max_of_left_or_right, lte_max_of_right, lte_min_iff, lte_min_imp, lte_min_of_bounds, max_assoc, max_eq_left_of_gt, max_eq_left_of_not_lt, max_eq_right_of_gt, max_eq_right_of_not_gt, max_gte_iff, max_gte_imp, max_gte_of_left_or_right, max_left_comm, max_lt_iff, max_lt_imp, max_lt_of_upper_bounds, max_lte_iff, max_lte_imp, max_lte_of_upper_bounds, max_of_gt, max_of_lt, max_right_comm, min_assoc, min_eq_left_of_lt, min_eq_left_of_not_gt, min_eq_right_of_lt, min_eq_right_of_not_lt, min_gte_iff, min_gte_imp, min_gte_of_bounds, min_left_comm, min_lt_iff, min_lt_of_left_or_right, min_lte_iff, min_lte_imp, min_lte_of_left, min_lte_of_left_or_right, min_lte_of_right, min_of_gt, min_of_lt, min_right_comm

theorem lte_iff_not_gt[L: LinearOrder](a: L, b: L) {
    a <= b = not (a > b)
} by {
    if a <= b {
        not_gt_imp_lte(a, b)
        not (a > b)
    }
    if not (a > b) {
        not_gt_imp_lte(a, b)
        a <= b
    }
    a <= b = not (a > b)
}

theorem gte_iff_not_lt[L: LinearOrder](a: L, b: L) {
    a >= b = not (a < b)
} by {
    if a >= b {
        not_lt_imp_gte(a, b)
        not (a < b)
    }
    if not (a < b) {
        not_lt_imp_gte(a, b)
        a >= b
    }
    a >= b = not (a < b)
}

theorem lt_iff_not_gte[L: LinearOrder](a: L, b: L) {
    a < b = not (a >= b)
} by {
    if a < b {
        a != b
        if a >= b {
            a = b
            false
        }
    }
    if not (a >= b) {
        not_gte_imp_lt(a, b)
        a < b
    }
    a < b = not (a >= b)
}

theorem gt_iff_not_lte[L: LinearOrder](a: L, b: L) {
    a > b = not (a <= b)
} by {
    if a > b {
        a != b
        if a <= b {
            a = b
            false
        }
    }
    if not (a <= b) {
        not_lte_imp_gt(a, b)
        a > b
    }
    a > b = not (a <= b)
}

theorem lte_of_not_gt[L: LinearOrder](a: L, b: L) {
    not (a > b) implies a <= b
} by {
    if not (a > b) {
        not_gt_imp_lte(a, b)
        a <= b
    }
}

theorem gte_of_not_lt[L: LinearOrder](a: L, b: L) {
    not (a < b) implies a >= b
} by {
    if not (a < b) {
        not_lt_imp_gte(a, b)
        a >= b
    }
}

theorem lt_of_not_gte[L: LinearOrder](a: L, b: L) {
    not (a >= b) implies a < b
} by {
    if not (a >= b) {
        not_gte_imp_lt(a, b)
        a < b
    }
}

theorem gt_of_not_lte[L: LinearOrder](a: L, b: L) {
    not (a <= b) implies a > b
} by {
    if not (a <= b) {
        not_lte_imp_gt(a, b)
        a > b
    }
}

theorem lt_or_gt_of_ne[L: LinearOrder](a: L, b: L) {
    a != b implies a < b or a > b
} by {
    if a != b {
        lte_or_gte(a, b)
        if a <= b {
            if a = b {
                false
            } else {
                a < b
                a < b or a > b
            }
        } else {
            a > b
            a < b or a > b
        }
    }
}

theorem gt_or_lt_of_ne[L: LinearOrder](a: L, b: L) {
    a != b implies a > b or a < b
} by {
    if a != b {
        lt_or_gt_of_ne(a, b)
        if a < b {
            a > b or a < b
        } else {
            a > b
            a > b or a < b
        }
    }
}

theorem lt_or_eq_or_gt[L: LinearOrder](a: L, b: L) {
    a < b or a = b or a > b
} by {
    if a = b {
        a < b or a = b or a > b
    } else {
        lt_or_gt_of_ne(a, b)
        if a < b {
            a < b or a = b or a > b
        } else {
            a > b
            a < b or a = b or a > b
        }
    }
}

/// Exactly one of the standard comparison cases holds up to disjunction.
theorem trichotomy[L: LinearOrder](a: L, b: L) {
    a < b or a = b or a > b
} by {
    lt_or_eq_or_gt(a, b)
}

theorem gt_or_eq_or_lt[L: LinearOrder](a: L, b: L) {
    a > b or a = b or a < b
} by {
    if a = b {
        a > b or a = b or a < b
    } else {
        gt_or_lt_of_ne(a, b)
        if a > b {
            a > b or a = b or a < b
        } else {
            a < b
            a > b or a = b or a < b
        }
    }
}

/// Trichotomy with equality as the final case.
theorem lt_or_gt_or_eq[L: LinearOrder](a: L, b: L) {
    a < b or a > b or a = b
} by {
    if a = b {
        a < b or a > b or a = b
    } else {
        lt_or_gt_of_ne(a, b)
        if a < b {
            a < b or a > b or a = b
        } else {
            a > b
            a < b or a > b or a = b
        }
    }
}

/// Trichotomy with the reverse strict comparison written literally.
theorem lt_or_lt_swap_or_eq[L: LinearOrder](a: L, b: L) {
    a < b or b < a or a = b
} by {
    lt_or_gt_or_eq(a, b)
}

/// Reverse trichotomy with equality as the final case.
theorem gt_or_lt_or_eq[L: LinearOrder](a: L, b: L) {
    a > b or a < b or a = b
} by {
    if a = b {
        a > b or a < b or a = b
    } else {
        gt_or_lt_of_ne(a, b)
        if a > b {
            a > b or a < b or a = b
        } else {
            a < b
            a > b or a < b or a = b
        }
    }
}

/// Reverse trichotomy with the reverse strict comparison written literally.
theorem lt_swap_or_lt_or_eq[L: LinearOrder](a: L, b: L) {
    b < a or a < b or a = b
} by {
    gt_or_lt_or_eq(a, b)
}

theorem eq_or_lt_or_gt[L: LinearOrder](a: L, b: L) {
    a = b or a < b or a > b
} by {
    if a = b {
        a = b or a < b or a > b
    } else {
        lt_or_gt_of_ne(a, b)
        if a < b {
            a = b or a < b or a > b
        } else {
            a > b
            a = b or a < b or a > b
        }
    }
}

theorem eq_or_gt_or_lt[L: LinearOrder](a: L, b: L) {
    a = b or a > b or a < b
} by {
    if a = b {
        a = b or a > b or a < b
    } else {
        gt_or_lt_of_ne(a, b)
        if a > b {
            a = b or a > b or a < b
        } else {
            a < b
            a = b or a > b or a < b
        }
    }
}

theorem ne_iff_lt_or_gt[L: LinearOrder](a: L, b: L) {
    a != b = (a < b or a > b)
} by {
    if a != b {
        lt_or_gt_of_ne(a, b)
        a < b or a > b
    }
    if a < b or a > b {
        if a < b {
            a != b
        } else {
            a != b
        }
    }
    a != b = (a < b or a > b)
}

theorem ne_iff_gt_or_lt[L: LinearOrder](a: L, b: L) {
    a != b = (a > b or a < b)
} by {
    if a != b {
        gt_or_lt_of_ne(a, b)
        a > b or a < b
    }
    if a > b or a < b {
        if a > b {
            a != b
        } else {
            a != b
        }
    }
    a != b = (a > b or a < b)
}

theorem eq_of_not_lt_of_not_gt[L: LinearOrder](a: L, b: L) {
    not (a < b) and not (a > b) implies a = b
} by {
    if not (a < b) and not (a > b) {
        not_lt_imp_gte(a, b)
        a >= b
        not_gt_imp_lte(a, b)
        a <= b
        lte_antisymm(a, b)
        a = b
    }
}

theorem eq_of_not_gt_of_not_lt[L: LinearOrder](a: L, b: L) {
    not (a > b) and not (a < b) implies a = b
} by {
    if not (a > b) and not (a < b) {
        eq_of_not_lt_of_not_gt(a, b)
        a = b
    }
}

theorem eq_iff_not_lt_and_not_gt[L: LinearOrder](a: L, b: L) {
    a = b = (not (a < b) and not (a > b))
} by {
    if a = b {
        if a < b {
            a != b
            false
        }
        if a > b {
            a != b
            false
        }
        not (a < b) and not (a > b)
    }
    if not (a < b) and not (a > b) {
        eq_of_not_lt_of_not_gt(a, b)
        a = b
    }
    a = b = (not (a < b) and not (a > b))
}

theorem eq_iff_not_gt_and_not_lt[L: LinearOrder](a: L, b: L) {
    a = b = (not (a > b) and not (a < b))
} by {
    if a = b {
        if a > b {
            a != b
            false
        }
        if a < b {
            a != b
            false
        }
        not (a > b) and not (a < b)
    }
    if not (a > b) and not (a < b) {
        eq_of_not_gt_of_not_lt(a, b)
        a = b
    }
    a = b = (not (a > b) and not (a < b))
}

theorem eq_or_lt_of_not_gt[L: LinearOrder](a: L, b: L) {
    not (a > b) implies a = b or a < b
} by {
    if not (a > b) {
        not_gt_imp_lte(a, b)
        eq_or_lt_of_lte(a, b)
        a = b or a < b
    }
}

theorem eq_or_gt_of_not_lt[L: LinearOrder](a: L, b: L) {
    not (a < b) implies a = b or a > b
} by {
    if not (a < b) {
        not_lt_imp_gte(a, b)
        eq_or_gt_of_gte(a, b)
        a = b or a > b
    }
}

theorem lt_or_eq_of_not_gt[L: LinearOrder](a: L, b: L) {
    not (a > b) implies a < b or a = b
} by {
    if not (a > b) {
        eq_or_lt_of_not_gt(a, b)
        if a = b {
            a < b or a = b
        } else {
            a < b
            a < b or a = b
        }
    }
}

theorem gt_or_eq_of_not_lt[L: LinearOrder](a: L, b: L) {
    not (a < b) implies a > b or a = b
} by {
    if not (a < b) {
        eq_or_gt_of_not_lt(a, b)
        if a = b {
            a > b or a = b
        } else {
            a > b
            a > b or a = b
        }
    }
}

theorem lte_or_eq_or_gt[L: LinearOrder](a: L, b: L) {
    a <= b or a = b or a > b
} by {
    lte_or_gt(a, b)
    if a <= b {
        a <= b or a = b or a > b
    } else {
        a > b
        a <= b or a = b or a > b
    }
}

theorem gte_or_eq_or_lt[L: LinearOrder](a: L, b: L) {
    a >= b or a = b or a < b
} by {
    gte_or_lt(a, b)
    if a >= b {
        a >= b or a = b or a < b
    } else {
        a < b
        a >= b or a = b or a < b
    }
}

theorem min_eq_left_iff_not_gt[L: LinearOrder](a: L, b: L) {
    a.min(b) = a = not (a > b)
} by {
    if a.min(b) = a {
        min_imp_lte(a, b)
        a.min(b) <= b
        a <= b
        not_gt_imp_lte(a, b)
        not (a > b)
    }
    if not (a > b) {
        min_eq_left_of_not_gt(a, b)
        a.min(b) = a
    }
    a.min(b) = a = not (a > b)
}

theorem min_eq_right_iff_not_lt[L: LinearOrder](a: L, b: L) {
    a.min(b) = b = not (a < b)
} by {
    if a.min(b) = b {
        min_imp_lte(a, b)
        a.min(b) <= a
        a >= b
        not_lt_imp_gte(a, b)
        not (a < b)
    }
    if not (a < b) {
        min_eq_right_of_not_lt(a, b)
        a.min(b) = b
    }
    a.min(b) = b = not (a < b)
}

theorem max_eq_left_iff_not_lt[L: LinearOrder](a: L, b: L) {
    a.max(b) = a = not (a < b)
} by {
    if a.max(b) = a {
        max_imp_gte(a, b)
        a.max(b) >= b
        a >= b
        not_lt_imp_gte(a, b)
        not (a < b)
    }
    if not (a < b) {
        max_eq_left_of_not_lt(a, b)
        a.max(b) = a
    }
    a.max(b) = a = not (a < b)
}

theorem max_eq_right_iff_not_gt[L: LinearOrder](a: L, b: L) {
    a.max(b) = b = not (a > b)
} by {
    if a.max(b) = b {
        max_imp_gte(a, b)
        a.max(b) >= a
        a <= b
        not_gt_imp_lte(a, b)
        not (a > b)
    }
    if not (a > b) {
        max_eq_right_of_not_gt(a, b)
        a.max(b) = b
    }
    a.max(b) = b = not (a > b)
}

theorem min_gt_imp[L: LinearOrder](a: L, b: L, c: L) {
    a.min(b) > c implies a > c and b > c
} by {
    if a.min(b) > c {
        lt_min_imp(c, a, b)
        c < a and c < b
        a > c
        b > c
        a > c and b > c
    }
}

theorem min_gt_of_bounds[L: LinearOrder](a: L, b: L, c: L) {
    a > c and b > c implies a.min(b) > c
} by {
    if a > c and b > c {
        lt_min_of_bounds(c, a, b)
        c < a.min(b)
        a.min(b) > c
    }
}

theorem min_gt_iff[L: LinearOrder](a: L, b: L, c: L) {
    a.min(b) > c = (a > c and b > c)
} by {
    min_gt_imp(a, b, c)
    min_gt_of_bounds(a, b, c)
    a.min(b) > c = (a > c and b > c)
}

theorem min_lt_imp[L: LinearOrder](a: L, b: L, c: L) {
    a.min(b) < c implies a < c or b < c
} by {
    if a.min(b) < c {
        min_is_one(a, b)
        if a.min(b) = a {
            a < c
            a < c or b < c
        } else {
            a.min(b) = b
            b < c
            a < c or b < c
        }
    }
}

theorem lt_max_imp[L: LinearOrder](c: L, a: L, b: L) {
    c < a.max(b) implies c < a or c < b
} by {
    if c < a.max(b) {
        max_is_one(a, b)
        if a.max(b) = a {
            c < a
            c < a or c < b
        } else {
            a.max(b) = b
            c < b
            c < a or c < b
        }
    }
}

theorem max_gt_imp[L: LinearOrder](a: L, b: L, c: L) {
    a.max(b) > c implies a > c or b > c
} by {
    if a.max(b) > c {
        lt_max_imp(c, a, b)
        c < a or c < b
        if c < a {
            a > c
            a > c or b > c
        } else {
            c < b
            b > c
            a > c or b > c
        }
    }
}

theorem max_gt_of_left_or_right[L: LinearOrder](a: L, b: L, c: L) {
    a > c or b > c implies a.max(b) > c
} by {
    if a > c or b > c {
        if a > c {
            lt_max_of_left_or_right(c, a, b)
            a.max(b) > c
        } else {
            b > c
            lt_max_of_left_or_right(c, a, b)
            a.max(b) > c
        }
    }
}

theorem max_gt_iff[L: LinearOrder](a: L, b: L, c: L) {
    a.max(b) > c = (a > c or b > c)
} by {
    max_gt_imp(a, b, c)
    max_gt_of_left_or_right(a, b, c)
    a.max(b) > c = (a > c or b > c)
}

theorem min_monotone_left[L: LinearOrder](a: L, b: L, c: L) {
    a <= b implies a.min(c) <= b.min(c)
} by {
    if a <= b {
        lte_min_iff(a.min(c), b, c)
        min_lte_left(a, c)
        lte_trans(a.min(c), a, b)
        a.min(c) <= b
        min_lte_right(a, c)
        a.min(c) <= c
        a.min(c) <= b.min(c)
    }
}

theorem min_monotone_right[L: LinearOrder](a: L, b: L, c: L) {
    a <= b implies c.min(a) <= c.min(b)
} by {
    if a <= b {
        min_symm(c, a)
        min_symm(c, b)
        min_monotone_left(a, b, c)
        c.min(a) <= c.min(b)
    }
}

theorem min_monotone[L: LinearOrder](a1: L, a2: L, b1: L, b2: L) {
    a1 <= a2 and b1 <= b2 implies a1.min(b1) <= a2.min(b2)
} by {
    if a1 <= a2 and b1 <= b2 {
        min_monotone_left(a1, a2, b1)
        a1.min(b1) <= a2.min(b1)
        min_monotone_right(b1, b2, a2)
        a2.min(b1) <= a2.min(b2)
        lte_trans(a1.min(b1), a2.min(b1), a2.min(b2))
        a1.min(b1) <= a2.min(b2)
    }
}

theorem max_monotone_left[L: LinearOrder](a: L, b: L, c: L) {
    a <= b implies a.max(c) <= b.max(c)
} by {
    if a <= b {
        max_lte_iff(a, c, b.max(c))
        lte_max_left(b, c)
        lte_trans(a, b, b.max(c))
        a <= b.max(c)
        lte_max_right(b, c)
        c <= b.max(c)
        a.max(c) <= b.max(c)
    }
}

theorem max_monotone_right[L: LinearOrder](a: L, b: L, c: L) {
    a <= b implies c.max(a) <= c.max(b)
} by {
    if a <= b {
        max_symm(c, a)
        max_symm(c, b)
        max_monotone_left(a, b, c)
        c.max(a) <= c.max(b)
    }
}

theorem max_monotone[L: LinearOrder](a1: L, a2: L, b1: L, b2: L) {
    a1 <= a2 and b1 <= b2 implies a1.max(b1) <= a2.max(b2)
} by {
    if a1 <= a2 and b1 <= b2 {
        max_monotone_left(a1, a2, b1)
        a1.max(b1) <= a2.max(b1)
        max_monotone_right(b1, b2, a2)
        a2.max(b1) <= a2.max(b2)
        lte_trans(a1.max(b1), a2.max(b1), a2.max(b2))
        a1.max(b1) <= a2.max(b2)
    }
}

theorem min_lt_left_of_lt[L: LinearOrder](a: L, b: L, c: L) {
    a < c implies a.min(b) < c
} by {
    if a < c {
        min_lte_left(a, b)
        lt_of_lte_of_lt(a.min(b), a, c)
        a.min(b) < c
    }
}

theorem min_lt_right_of_lt[L: LinearOrder](a: L, b: L, c: L) {
    b < c implies a.min(b) < c
} by {
    if b < c {
        min_lte_right(a, b)
        lt_of_lte_of_lt(a.min(b), b, c)
        a.min(b) < c
    }
}

theorem lt_max_left_of_lt[L: LinearOrder](c: L, a: L, b: L) {
    c < a implies c < a.max(b)
} by {
    if c < a {
        lte_max_left(a, b)
        lt_of_lt_of_lte(c, a, a.max(b))
        c < a.max(b)
    }
}

theorem lt_max_right_of_lt[L: LinearOrder](c: L, a: L, b: L) {
    c < b implies c < a.max(b)
} by {
    if c < b {
        lte_max_right(a, b)
        lt_of_lt_of_lte(c, b, a.max(b))
        c < a.max(b)
    }
}

theorem min_lte_max[L: LinearOrder](a: L, b: L) {
    a.min(b) <= a.max(b)
} by {
    lte_max_iff(a.min(b), a, b)
    min_lte_left(a, b)
    min_lte_right(a, b)
    a.min(b) <= a.max(b)
}

theorem min_min_left[L: LinearOrder](a: L, b: L) {
    a.min(a.min(b)) = a.min(b)
} by {
    min_assoc(a, a, b)
    min_idem(a)
}

theorem min_min_right[L: LinearOrder](a: L, b: L) {
    a.min(b.min(b)) = a.min(b)
} by {
    min_idem(b)
}

theorem max_max_left[L: LinearOrder](a: L, b: L) {
    a.max(a.max(b)) = a.max(b)
} by {
    max_assoc(a, a, b)
    max_idem(a)
}

theorem max_max_right[L: LinearOrder](a: L, b: L) {
    a.max(b.max(b)) = a.max(b)
} by {
    max_idem(b)
}
