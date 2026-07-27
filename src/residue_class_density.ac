from int import Int
from list import List
from nat import Nat
from pair import Pair
from rat import Rat, lte_trans
from number_theory import congruence_class_contains, covers_int

numerals Rat

/// The rational reciprocal of a modulus.
///
/// The reciprocal of zero is zero by the convention on `Rat.inverse`, which is the right
/// convention here: a class with modulus zero is a single integer and contributes nothing to
/// the density.
define modulus_weight(m: Nat) -> Rat {
    Rat.from_int(Int.from_nat(m)).inverse
}

/// The density of a residue-class system.
///
/// The sum of the reciprocals of the moduli. A covering system has density at least one and a
/// disjoint system at most one, so the two conditions together pin the density exactly.
define system_density(system: List[Pair[Nat, Nat]]) -> Rat {
    match system {
        List.nil {
            Rat.0
        }
        List.cons(head, tail) {
            modulus_weight(head.first) + system_density(tail)
        }
    }
}

/// The empty system has density zero.
theorem system_density_nil {
    system_density(List.nil[Pair[Nat, Nat]]) = Rat.0
}

/// Adding a class adds the reciprocal of its modulus.
theorem system_density_cons(head: Pair[Nat, Nat], tail: List[Pair[Nat, Nat]]) {
    system_density(List.cons(head, tail)) =
        modulus_weight(head.first) + system_density(tail)
}

/// A class of modulus one carries all the density.
theorem modulus_weight_one {
    modulus_weight(Nat.1) = Rat.1
} by {
    Int.from_nat(Nat.1) = Int.1
    Rat.from_int(Int.1) = Rat.1
    modulus_weight(Nat.1) = Rat.1.inverse
    Rat.1.inverse = Rat.1
}

/// A class of modulus zero carries no density.
///
/// Such a class contains a single integer, so it should contribute nothing, and the
/// convention that the inverse of zero is zero gives exactly that.
theorem modulus_weight_zero {
    modulus_weight(Nat.0) = Rat.0
} by {
    Int.from_nat(Nat.0) = Int.0
    Rat.from_int(Int.0) = Rat.0
    modulus_weight(Nat.0) = Rat.0.inverse
    Rat.0.inverse = Rat.0
}

/// Every weight is nonnegative.
theorem modulus_weight_nonnegative(m: Nat) {
    not modulus_weight(m).is_negative
} by {
    if m = Nat.0 {
        modulus_weight_zero
        modulus_weight(m) = Rat.0
        not Rat.0.is_negative
        not modulus_weight(m).is_negative
    }
    if m != Nat.0 {
        Int.from_nat(m).is_positive
        Rat.from_int(Int.from_nat(m)).is_positive
        Rat.from_int(Int.from_nat(m)).inverse.is_positive
        modulus_weight(m).is_positive
        not modulus_weight(m).is_negative
    }
    not modulus_weight(m).is_negative
}

/// The density of a system is at least the density of its tail.
///
/// Weights are never negative, so dropping a class can only lower the density. This is the
/// monotonicity that makes the density bound on a disjoint system usable after a class is
/// removed.
theorem system_density_cons_ge_tail(
    head: Pair[Nat, Nat], tail: List[Pair[Nat, Nat]]
) {
    system_density(tail) <= system_density(List.cons(head, tail))
} by {
    modulus_weight_nonnegative(head.first)
    not modulus_weight(head.first).is_negative
    Rat.0 <= modulus_weight(head.first)
    system_density(tail) <= modulus_weight(head.first) + system_density(tail)
    system_density_cons(head, tail)
    system_density(tail) <= system_density(List.cons(head, tail))
}

/// Every density is nonnegative.
theorem system_density_nonnegative(system: List[Pair[Nat, Nat]]) {
    Rat.0 <= system_density(system)
} by {
    define p(s: List[Pair[Nat, Nat]]) -> Bool {
        Rat.0 <= system_density(s)
    }
    system_density_nil
    Rat.0 <= Rat.0
    p(List.nil[Pair[Nat, Nat]])
    forall(head: Pair[Nat, Nat], tail: List[Pair[Nat, Nat]]) {
        if p(tail) {
            Rat.0 <= system_density(tail)
            system_density_cons_ge_tail(head, tail)
            system_density(tail) <= system_density(List.cons(head, tail))
            lte_trans(Rat.0, system_density(tail), system_density(List.cons(head, tail)))
            Rat.0 <= system_density(List.cons(head, tail))
            p(List.cons(head, tail))
        }
        (p(tail) implies p(List.cons(head, tail)))
    }
    p(List.nil[Pair[Nat, Nat]]) and forall(head: Pair[Nat, Nat], tail: List[Pair[Nat, Nat]]) {
        p(tail) implies p(List.cons(head, tail))
    }
    List.induction(p)
    p(system)
}

/// A system containing a class of modulus one already has density at least one.
///
/// The trivial covering system consisting of that one class meets the covering bound exactly,
/// which is why the standard questions about covering systems require the moduli to exceed
/// one.
theorem system_density_ge_one_of_modulus_one(tail: List[Pair[Nat, Nat]], r: Nat) {
    Rat.1 <= system_density(List.cons(Pair.new(Nat.1, r), tail))
} by {
    Pair.new(Nat.1, r).first = Nat.1
    modulus_weight_one
    modulus_weight(Pair.new(Nat.1, r).first) = Rat.1
    system_density_cons(Pair.new(Nat.1, r), tail)
    system_density(List.cons(Pair.new(Nat.1, r), tail)) = Rat.1 + system_density(tail)
    system_density_nonnegative(tail)
    Rat.0 <= system_density(tail)
    Rat.1 <= Rat.1 + system_density(tail)
    Rat.1 <= system_density(List.cons(Pair.new(Nat.1, r), tail))
}
