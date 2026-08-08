from comm_ring import CommRing
from data.basic.set import Set, indexed_intersection, indexed_intersection_contains_at,
    indexed_intersection_contains_of_forall, indexed_union_complement_eq_intersection_complement,
    indexed_union, set_ext
from algebra.ring.ideal import Ideal, bundled_ideal_subset
from algebra.ring.ideal_indexed_sum import ideal_indexed_sum, ideal_indexed_sum_subset_iff
from algebra.ring.spec import Spec, spec_vanishing_contains_eq, spec_vanishing_family,
    spec_zariski_open

/// Vanishing of an indexed ideal sum is the indexed intersection of vanishings.
theorem spec_vanishing_indexed_sum[R: CommRing, I](family: I -> Ideal[R]) {
    Spec[R].vanishing(ideal_indexed_sum(family)) = indexed_intersection(spec_vanishing_family(family))
} by {
    forall(p: Spec[R]) {
        if Spec[R].vanishing(ideal_indexed_sum(family)).contains(p) {
            spec_vanishing_contains_eq(ideal_indexed_sum(family), p)
            bundled_ideal_subset(ideal_indexed_sum(family), p.point.as_ideal)
            ideal_indexed_sum_subset_iff(family, p.point.as_ideal)
            forall(i: I) {
                bundled_ideal_subset(family(i), p.point.as_ideal)
                spec_vanishing_contains_eq(family(i), p)
                Spec[R].vanishing(family(i)).contains(p)
                spec_vanishing_family(family, i) = Spec[R].vanishing(family(i))
                spec_vanishing_family(family)(i).contains(p)
            }
            indexed_intersection_contains_of_forall(spec_vanishing_family(family), p)
            indexed_intersection(spec_vanishing_family(family)).contains(p)
        }
        if indexed_intersection(spec_vanishing_family(family)).contains(p) {
            forall(i: I) {
                indexed_intersection_contains_at(spec_vanishing_family(family), i, p)
                spec_vanishing_family(family)(i).contains(p)
                spec_vanishing_family(family, i) = Spec[R].vanishing(family(i))
                Spec[R].vanishing(family(i)).contains(p)
                spec_vanishing_contains_eq(family(i), p)
                bundled_ideal_subset(family(i), p.point.as_ideal)
            }
            ideal_indexed_sum_subset_iff(family, p.point.as_ideal)
            bundled_ideal_subset(ideal_indexed_sum(family), p.point.as_ideal)
            spec_vanishing_contains_eq(ideal_indexed_sum(family), p)
            Spec[R].vanishing(ideal_indexed_sum(family)).contains(p)
        }
        Spec[R].vanishing(ideal_indexed_sum(family)).contains(p) =
            indexed_intersection(spec_vanishing_family(family)).contains(p)
    }
    set_ext(Spec[R].vanishing(ideal_indexed_sum(family)), indexed_intersection(spec_vanishing_family(family)))
}

/// Complements of an indexed family of vanishing sets.
define spec_vanishing_compl_family[R: CommRing, I](family: I -> Ideal[R], i: I) -> Set[Spec[R]] {
    Spec[R].vanishing(family(i)).c
}

/// The complement of `V(sum_i I_i)` is the indexed union of the complements `V(I_i).c`.
theorem spec_vanishing_indexed_sum_compl[R: CommRing, I](family: I -> Ideal[R]) {
    Spec[R].vanishing(ideal_indexed_sum(family)).c =
        indexed_union(spec_vanishing_compl_family(family))
} by {
    spec_vanishing_indexed_sum(family)
    indexed_union_complement_eq_intersection_complement(spec_vanishing_family(family))
    indexed_union(function(i: I) { spec_vanishing_family(family)(i).c }) =
        indexed_intersection(spec_vanishing_family(family)).c
    forall(i: I) {
        spec_vanishing_compl_family(family, i) = Spec[R].vanishing(family(i)).c
        spec_vanishing_family(family, i) = Spec[R].vanishing(family(i))
        spec_vanishing_compl_family(family)(i) = spec_vanishing_family(family)(i).c
    }
    spec_vanishing_compl_family(family) =
        function(i: I) { spec_vanishing_family(family)(i).c }
    indexed_union(spec_vanishing_compl_family(family)) =
        indexed_union(function(i: I) { spec_vanishing_family(family)(i).c })
    indexed_union(spec_vanishing_compl_family(family)) =
        indexed_intersection(spec_vanishing_family(family)).c
    Spec[R].vanishing(ideal_indexed_sum(family)).c =
        indexed_union(spec_vanishing_compl_family(family))
}

/// The indexed union of complements of vanishing sets is Zariski-open.
theorem spec_zariski_open_indexed_union_vanishing_compl[R: CommRing, I](family: I -> Ideal[R]) {
    spec_zariski_open(indexed_union(spec_vanishing_compl_family(family)))
} by {
    spec_vanishing_indexed_sum_compl(family)
    indexed_union(spec_vanishing_compl_family(family)) =
        Spec[R].vanishing(ideal_indexed_sum(family)).c
    exists(i: Ideal[R]) {
        indexed_union(spec_vanishing_compl_family(family)) = Spec[R].vanishing(i).c
    }
}
