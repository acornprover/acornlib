from finite_set import FiniteSet, fs_union
from data.basic.logic import exists_intro
from list import List
from multiset import Multiset, add_multiplicity, filter_multiplicity_false,
    filter_multiplicity_true, insert_multiplicity_other, insert_multiplicity_self,
    multiset_ext, singleton_multiplicity_other, singleton_multiplicity_self
from multiset_list import from_list_has_finite_support_witness, from_list_singleton
from nat import Nat, add_to_zero
from data.basic.set import not_union_contains_eq
numerals Nat

attributes Multiset[T] {
    /// True if the multiset has a finite support set.
    define has_finite_support(self) -> Bool {
        exists(support: FiniteSet[T]) {
            forall(x: T) {
                not support.contains(x) implies self.multiplicity(x) = Nat.0
            }
        }
    }
}

/// The empty multiset has finite support.
theorem multiset_empty_has_finite_support[T] {
    Multiset.empty[T].has_finite_support
} by {
    forall(x: T) {
        if not FiniteSet.empty[T].contains(x) {
            Multiset.empty[T].multiplicity(x) = Nat.0
        }
    }
    exists_intro(function(support: FiniteSet[T]) {
        forall(x: T) {
            not support.contains(x) implies Multiset.empty[T].multiplicity(x) = Nat.0
        }
    }, FiniteSet.empty[T])
}

/// Every multiset obtained from a list has finite support.
theorem multiset_from_list_has_finite_support[T](items: List[T]) {
    Multiset.from_list(items).has_finite_support
} by {
    from_list_has_finite_support_witness(items)
}

/// A singleton multiset has finite support.
theorem multiset_singleton_has_finite_support[T](item: T) {
    Multiset.singleton(item).has_finite_support
} by {
    from_list_singleton(item)
    multiset_from_list_has_finite_support(List.singleton(item))
}

/// Inserting an element is the same as adding the corresponding singleton multiset.
theorem multiset_insert_eq_add_singleton[T](a: Multiset[T], item: T) {
    a.insert(item) = a + Multiset.singleton(item)
} by {
    forall(x: T) {
        if item = x {
            insert_multiplicity_self(a, item)
            add_multiplicity(a, Multiset.singleton(item), x)
            singleton_multiplicity_self(item)
            a.multiplicity(x) + Nat.1 = a.multiplicity(x).suc
            a.insert(item).multiplicity(x) = (a + Multiset.singleton(item)).multiplicity(x)
        } else {
            insert_multiplicity_other(a, item, x)
            add_multiplicity(a, Multiset.singleton(item), x)
            singleton_multiplicity_other(item, x)
            a.insert(item).multiplicity(x) = (a + Multiset.singleton(item)).multiplicity(x)
        }
    }
    multiset_ext(a.insert(item), a + Multiset.singleton(item))
}

/// The sum of multisets with displayed finite supports is supported in the union of those supports.
theorem multiset_add_has_support_in_union[T](a: Multiset[T], b: Multiset[T],
        support_a: FiniteSet[T], support_b: FiniteSet[T]) {
    (forall(x: T) { not support_a.contains(x) implies a.multiplicity(x) = Nat.0 }) and
    (forall(x: T) { not support_b.contains(x) implies b.multiplicity(x) = Nat.0 })
    implies forall(x: T) {
        not fs_union(support_a, support_b).contains(x) implies (a + b).multiplicity(x) = Nat.0
    }
} by {
    if (forall(x: T) { not support_a.contains(x) implies a.multiplicity(x) = Nat.0 }) and
        (forall(x: T) { not support_b.contains(x) implies b.multiplicity(x) = Nat.0 }) {
        forall(x: T) {
            if not fs_union(support_a, support_b).contains(x) {
                fs_union(support_a, support_b).underlying_set =
                    support_a.underlying_set.union(support_b.underlying_set)
                fs_union(support_a, support_b).contains(x) =
                    support_a.underlying_set.union(support_b.underlying_set).contains(x)
                not support_a.underlying_set.union(support_b.underlying_set).contains(x)
                not_union_contains_eq[T](support_a.underlying_set, support_b.underlying_set, x)
                not support_a.underlying_set.contains(x) and not support_b.underlying_set.contains(x)
                not support_a.contains(x)
                not support_b.contains(x)
                not support_a.contains(x) implies a.multiplicity(x) = Nat.0
                not support_b.contains(x) implies b.multiplicity(x) = Nat.0
                a.multiplicity(x) = Nat.0
                b.multiplicity(x) = Nat.0
                add_multiplicity(a, b, x)
                (a + b).multiplicity(x) = a.multiplicity(x) + b.multiplicity(x)
                (a + b).multiplicity(x) = Nat.0 + Nat.0
                Nat.0 + Nat.0 = Nat.0
                (a + b).multiplicity(x) = Nat.0
            }
        }
    }
}

/// Pointwise multiset addition preserves finite support.
theorem multiset_add_preserves_finite_support[T](a: Multiset[T], b: Multiset[T]) {
    a.has_finite_support and b.has_finite_support implies (a + b).has_finite_support
} by {
    if a.has_finite_support and b.has_finite_support {
        let support_a: FiniteSet[T] satisfy {
            forall(x: T) {
                not support_a.contains(x) implies a.multiplicity(x) = Nat.0
            }
        }
        let support_b: FiniteSet[T] satisfy {
            forall(x: T) {
                not support_b.contains(x) implies b.multiplicity(x) = Nat.0
            }
        }
        forall(x: T) {
            if not fs_union(support_a, support_b).contains(x) {
                fs_union(support_a, support_b).contains(x) =
                    support_a.underlying_set.union(support_b.underlying_set).contains(x)
                not_union_contains_eq[T](support_a.underlying_set, support_b.underlying_set, x)
                not support_a.contains(x)
                not support_b.contains(x)
                not support_a.contains(x) implies a.multiplicity(x) = Nat.0
                not support_b.contains(x) implies b.multiplicity(x) = Nat.0
                a.multiplicity(x) = Nat.0
                b.multiplicity(x) = Nat.0
                add_multiplicity(a, b, x)
                (a + b).multiplicity(x) = Nat.0
            }
        }
        exists_intro(function(support: FiniteSet[T]) {
            forall(x: T) {
                not support.contains(x) implies (a + b).multiplicity(x) = Nat.0
            }
        }, fs_union(support_a, support_b))
        (a + b).has_finite_support
    }
}

/// Finite support of a multiset sum is equivalent to finite support of both summands.
theorem multiset_add_has_finite_support_iff[T](a: Multiset[T], b: Multiset[T]) {
    (a + b).has_finite_support = (a.has_finite_support and b.has_finite_support)
} by {
    if (a + b).has_finite_support {
        let support: FiniteSet[T] satisfy {
            forall(x: T) {
                not support.contains(x) implies (a + b).multiplicity(x) = Nat.0
            }
        }
        forall(x: T) {
            if not support.contains(x) {
                (a + b).multiplicity(x) = Nat.0
                add_multiplicity(a, b, x)
                (a + b).multiplicity(x) = a.multiplicity(x) + b.multiplicity(x)
                a.multiplicity(x) + b.multiplicity(x) = Nat.0
                add_to_zero(a.multiplicity(x), b.multiplicity(x))
                a.multiplicity(x) = Nat.0
            }
        }
        exists_intro(function(s: FiniteSet[T]) {
            forall(x: T) {
                not s.contains(x) implies a.multiplicity(x) = Nat.0
            }
        }, support)
        exists(s: FiniteSet[T]) {
            forall(x: T) {
                not s.contains(x) implies a.multiplicity(x) = Nat.0
            }
        }
        a.has_finite_support
        forall(x: T) {
            if not support.contains(x) {
                (a + b).multiplicity(x) = Nat.0
                add_multiplicity(a, b, x)
                (a + b).multiplicity(x) = a.multiplicity(x) + b.multiplicity(x)
                a.multiplicity(x) + b.multiplicity(x) = Nat.0
                add_to_zero(a.multiplicity(x), b.multiplicity(x))
                b.multiplicity(x) = Nat.0
            }
        }
        exists_intro(function(s: FiniteSet[T]) {
            forall(x: T) {
                not s.contains(x) implies b.multiplicity(x) = Nat.0
            }
        }, support)
        exists(s: FiniteSet[T]) {
            forall(x: T) {
                not s.contains(x) implies b.multiplicity(x) = Nat.0
            }
        }
        b.has_finite_support
        a.has_finite_support and b.has_finite_support
    }
    if a.has_finite_support and b.has_finite_support {
        multiset_add_preserves_finite_support(a, b)
        (a + b).has_finite_support
    }
}

/// Inserting an element preserves finite support.
theorem multiset_insert_preserves_finite_support[T](a: Multiset[T], item: T) {
    a.has_finite_support implies a.insert(item).has_finite_support
} by {
    if a.has_finite_support {
        multiset_singleton_has_finite_support(item)
        multiset_add_preserves_finite_support(a, Multiset.singleton(item))
        multiset_insert_eq_add_singleton(a, item)
        a.insert(item).has_finite_support
    }
}

/// Filtering a finitely supported multiset preserves finite support.
theorem multiset_filter_preserves_finite_support[T](a: Multiset[T], pred: T -> Bool) {
    a.has_finite_support implies a.filter(pred).has_finite_support
} by {
    if a.has_finite_support {
        let support: FiniteSet[T] satisfy {
            forall(x: T) {
                not support.contains(x) implies a.multiplicity(x) = Nat.0
            }
        }
        forall(x: T) {
            if not support.contains(x) {
                a.multiplicity(x) = Nat.0
                if pred(x) {
                    filter_multiplicity_true(a, pred, x)
                    a.filter(pred).multiplicity(x) = a.multiplicity(x)
                    a.filter(pred).multiplicity(x) = Nat.0
                }
                if not pred(x) {
                    filter_multiplicity_false(a, pred, x)
                    a.filter(pred).multiplicity(x) = Nat.0
                }
                a.filter(pred).multiplicity(x) = Nat.0
            }
        }
        exists_intro(function(s: FiniteSet[T]) {
            forall(x: T) {
                not s.contains(x) implies a.filter(pred).multiplicity(x) = Nat.0
            }
        }, support)
        exists(s: FiniteSet[T]) {
            forall(x: T) {
                not s.contains(x) implies a.filter(pred).multiplicity(x) = Nat.0
            }
        }
        a.filter(pred).has_finite_support
    }
}
