from nat import Nat, from_nat
from int import Int
from rat import Rat, nat_lt_imp_rat_lt
from list import List
from pair import Pair
from real import Real, from_rat_maintains_lte, real_lte_imp_rat_lte, zero_inverse,
    real_from_rat_inverse, from_nat_is_from_rat
from residue_class_density import modulus_weight, system_density, system_density_nil,
    system_density_cons, modulus_weight_zero
from data.nat.nat_covering_density import system_density_real, system_density_real_nil,
    system_density_real_cons, system_reduced
from data.nat.nat_covering_bridge import is_covering_system, covering_system_density_ge_one
from data.nat.nat_disjoint_system import system_disjoint_nat, disjoint_system_density_le_one

/// The rational reciprocal of a modulus, embedded in the reals, is the real reciprocal.
///
/// Both sides send a modulus of zero to zero: the rational inverse of zero is zero by
/// convention, and dividing by zero in the reals gives zero for the same reason. So no
/// hypothesis on the modulus is needed.
theorem real_from_rat_modulus_weight(m: Nat) {
    Real.from_rat(modulus_weight(m)) = Real.1 / from_nat[Real](m)
} by {
    (Rat.from_nat(m) = Rat.from_int(Int.from_nat(m)))
    (modulus_weight(m) = Rat.from_nat(m).inverse)
    if m = Nat.0 {
        modulus_weight_zero
        modulus_weight(Nat.0) = Rat.0
        modulus_weight(m) = Rat.0
        (Real.0 = Real.from_rat(Rat.0))
        Real.from_rat(modulus_weight(m)) = Real.0
        (from_nat[Real](Nat.0) = Real.0)
        from_nat[Real](m) = Real.0
        (Real.1 / Real.0 = Real.1 * Real.0.inverse)
        zero_inverse
        Real.0.inverse = Real.0
        (Real.1 * Real.0 = Real.0)
        Real.1 / from_nat[Real](m) = Real.0
        Real.from_rat(modulus_weight(m)) = Real.1 / from_nat[Real](m)
    }
    if m != Nat.0 {
        Nat.0 < m
        nat_lt_imp_rat_lt(Nat.0, m)
        Rat.from_nat(Nat.0) < Rat.from_nat(m)
        (Rat.from_nat(Nat.0) = Rat.0)
        Rat.0 < Rat.from_nat(m)
        Rat.from_nat(m) != Rat.0
        real_from_rat_inverse(Rat.from_nat(m))
        (Real.from_rat(Rat.from_nat(m).inverse) = Real.from_rat(Rat.from_nat(m)).inverse)
        from_nat_is_from_rat(m)
        (from_nat[Real](m) = Real.from_rat(Rat.from_nat(m)))
        Real.from_rat(modulus_weight(m)) = from_nat[Real](m).inverse
        (Real.1 / from_nat[Real](m) = Real.1 * from_nat[Real](m).inverse)
        (Real.1 * from_nat[Real](m).inverse = from_nat[Real](m).inverse)
        Real.from_rat(modulus_weight(m)) = Real.1 / from_nat[Real](m)
    }
}

/// The two forms of the density of a system agree.
///
/// `system_density` sums the reciprocals over the rationals and `system_density_real` over the
/// reals; the embedding carries one sum to the other class by class.
theorem system_density_real_eq_from_rat(system: List[Pair[Nat, Nat]]) {
    system_density_real(system) = Real.from_rat(system_density(system))
} by {
    define p(s: List[Pair[Nat, Nat]]) -> Bool {
        system_density_real(s) = Real.from_rat(system_density(s))
    }
    system_density_nil
    (system_density(List.nil[Pair[Nat, Nat]]) = Rat.0)
    system_density_real_nil
    (system_density_real(List.nil[Pair[Nat, Nat]]) = Real.0)
    (Real.0 = Real.from_rat(Rat.0))
    p(List.nil[Pair[Nat, Nat]])
    forall(head: Pair[Nat, Nat], tail: List[Pair[Nat, Nat]]) {
        if p(tail) {
            system_density_real(tail) = Real.from_rat(system_density(tail))
            system_density_cons(head, tail)
            (system_density(List.cons(head, tail))
                = modulus_weight(head.first) + system_density(tail))
            (Real.from_rat(modulus_weight(head.first) + system_density(tail))
                = Real.from_rat(modulus_weight(head.first))
                    + Real.from_rat(system_density(tail)))
            real_from_rat_modulus_weight(head.first)
            (Real.from_rat(modulus_weight(head.first))
                = Real.1 / from_nat[Real](head.first))
            (Real.from_rat(system_density(List.cons(head, tail)))
                = Real.1 / from_nat[Real](head.first) + system_density_real(tail))
            system_density_real_cons(head, tail)
            (system_density_real(List.cons(head, tail))
                = Real.1 / from_nat[Real](head.first) + system_density_real(tail))
            p(List.cons(head, tail))
        }
        (p(tail) implies p(List.cons(head, tail)))
    }
    p(List.nil[Pair[Nat, Nat]]) and forall(h: Pair[Nat, Nat], t: List[Pair[Nat, Nat]]) {
        p(t) implies p(List.cons(h, t))
    }
    List.induction(p)
    p(system)
}

/// The reciprocals of the moduli of a covering system sum to at least one, over the rationals.
///
/// The same statement as `covering_system_density_ge_one`, read through the rational density
/// the system carries.
theorem covering_system_rat_density_ge_one(system: List[Pair[Nat, Nat]]) {
    system_reduced(system) and is_covering_system(system)
        implies Rat.1 <= system_density(system)
} by {
    if system_reduced(system) and is_covering_system(system) {
        covering_system_density_ge_one(system)
        Real.1 <= system_density_real(system)
        system_density_real_eq_from_rat(system)
        (system_density_real(system) = Real.from_rat(system_density(system)))
        (Real.1 = Real.from_rat(Rat.1))
        Real.from_rat(Rat.1) <= Real.from_rat(system_density(system))
        real_lte_imp_rat_lte(Rat.1, system_density(system))
        Rat.1 <= system_density(system)
    }
}

/// The reciprocals of the moduli of a disjoint system sum to at most one, over the rationals.
theorem disjoint_system_rat_density_le_one(system: List[Pair[Nat, Nat]]) {
    system_reduced(system) and system_disjoint_nat(system)
        implies system_density(system) <= Rat.1
} by {
    if system_reduced(system) and system_disjoint_nat(system) {
        disjoint_system_density_le_one(system)
        system_density_real(system) <= Real.1
        system_density_real_eq_from_rat(system)
        (system_density_real(system) = Real.from_rat(system_density(system)))
        (Real.1 = Real.from_rat(Rat.1))
        Real.from_rat(system_density(system)) <= Real.from_rat(Rat.1)
        real_lte_imp_rat_lte(system_density(system), Rat.1)
        system_density(system) <= Rat.1
    }
}

/// A system that is both covering and disjoint has density exactly one.
///
/// The two bounds meet. This is the exact statement the covering and disjoint conditions
/// together pin down, and the reason the sum of reciprocals is the right measure of a system.
theorem covering_disjoint_system_density(system: List[Pair[Nat, Nat]]) {
    system_reduced(system) and is_covering_system(system)
        and system_disjoint_nat(system) implies system_density(system) = Rat.1
} by {
    if system_reduced(system) and is_covering_system(system)
        and system_disjoint_nat(system) {
        covering_system_rat_density_ge_one(system)
        Rat.1 <= system_density(system)
        disjoint_system_rat_density_le_one(system)
        system_density(system) <= Rat.1
        system_density(system) = Rat.1
    }
}
