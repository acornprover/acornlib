from data.basic.functions import identity_fn
from nat import Nat, alt_induction, lt_not_ref, lte_trans, suc_ne, lt_suc, lt_imp_lt_suc
numerals Nat

/// A generic list data structure that can hold elements of any type.
/// Lists are constructed using nil (empty list) and cons (prepending an element).
inductive List[T] {
    /// The empty list.
    nil
    /// Constructs a list by prepending an element to an existing list.
    cons(T, List[T])
}

attributes List[T] {
    /// Concatenates two lists together.
    define add(self, other: List[T]) -> List[T] {
        match self {
            List.nil {
                other
            }
            List.cons(head, tail) {
                List.cons(head, tail.add(other))
            }
        }
    }

    /// True if this list contains the given item.
    define contains(self, item: T) -> Bool {
        match self {
            List.nil {
                false
            }
            List.cons(head, tail) {
                if head = item {
                    true
                } else {
                    tail.contains(item)
                }
            }
        }
    }
}

theorem add_nil[T](list: List[T]) {
    list + List.nil[T] = list
} by {
    define f(x: List[T]) -> Bool {
        x + List.nil[T] = x
    }
    f(List.nil)
    forall(head: T, tail: List[T]) {
        if f(tail) {
            f(List.cons(head, tail))
        }
    }
}

theorem nil_add[T](list: List[T]) {
    List.nil[T] + list = list
}

theorem add_contains_left[T](left: List[T], right: List[T], item: T) {
    left.contains(item) implies (left + right).contains(item)
} by {
    define f(x: List[T]) -> Bool {
        x.contains(item) implies (x + right).contains(item)
    }
    f(List.nil)
    forall(head: T, tail: List[T]) {
        if f(tail) {
            if List.cons(head, tail).contains(item) {
                if head = item {
                    (List.cons(head, tail) + right).contains(item)
                } else {
                    tail.contains(item)
                    (tail + right).contains(item)
                    (List.cons(head, tail) + right).contains(item)
                }
            }
            f(List.cons(head, tail))
        }
    }
}

theorem add_contains_right[T](left: List[T], right: List[T], item: T) {
    right.contains(item) implies (left + right).contains(item)
} by {
    define f(x: List[T]) -> Bool {
        (x + right).contains(item)
    }
    f(List.nil)
    forall(head: T, tail: List[T]) {
        if f(tail) {
            f(List.cons(head, tail))
        }
    }
}

theorem not_contains_add[T](left: List[T], right: List[T], item: T) {
    not left.contains(item) and not right.contains(item) implies not (left + right).contains(item)
} by {
    define f(x: List[T]) -> Bool {
        not x.contains(item) implies not (x + right).contains(item)
    }
    f(List.nil)
    forall(head: T, tail: List[T]) {
        if f(tail) {
            f(List.cons(head, tail))
        }
    }
}

/// Converse of `not_contains_add`: membership in a concatenation comes from
/// membership in one of the parts.
theorem add_contains_or[T](left: List[T], right: List[T], item: T) {
    (left + right).contains(item) implies left.contains(item) or right.contains(item)
} by {
    if (left + right).contains(item) {
        if not (left.contains(item) or right.contains(item)) {
            not_contains_add(left, right, item)
            false
        }
    }
}

attributes List[T] {
    /// True if this list contains every element of type T.
    define contains_every(self) -> Bool {
        forall(x: T) {
            self.contains(x)
        }
    }

    /// Yields the number of elements in the list.
    define length(self) -> Nat {
        match self {
            List.nil {
                Nat.0
            }
            List.cons(_, tail) {
                tail.length.suc
            }
        }
    }
}

theorem add_length[T](left: List[T], right: List[T]) {
    left.length + right.length = (left + right).length
} by {
    define f(x: List[T]) -> Bool {
        x.length + right.length = (x + right).length
    }
    f(List.nil)
    forall(head: T, tail: List[T]) {
        if f(tail) {
            f(List.cons(head, tail))
        }
    }
}

attributes List[T] {
    /// Creates a list containing a single element.
    let singleton: T -> List[T] = function(x: T) {
        List.cons(x, List.nil[T])
    }

    /// Removes all duplicate elements from the list.
    /// When duplicates exist, the last occurrence is kept.
    define unique(self) -> List[T] {
        match self {
            List.nil {
                List.nil[T]
            }
            List.cons(head, tail) {
                if tail.contains(head) {
                    tail.unique
                } else {
                    List.cons(head, tail.unique)
                }
            }
        }
    }

    /// True if all elements in the list are distinct.
    define is_unique(self) -> Bool {
        self.unique = self
    }
}

/// Membership in a singleton list determines equality with its element.
theorem singleton_contains_imp_eq[T](item: T, x: T) {
    List.singleton(item).contains(x) implies x = item
} by {
    if List.singleton(item).contains(x) {
        if item = x {
            x = item
        } else {
            not List.cons(item, List.nil[T]).contains(x)
            false
        }
    }
}

theorem singleton_unique[T](item: T) {
    List.singleton(item).is_unique
} by {
    not List.nil[T].contains(item)
}

theorem unique_length[T](list: List[T]) {
    list.unique.length <= list.length
} by {
    define f(x: List[T]) -> Bool {
        x.unique.length <= x.length
    }
    f(List.nil)
    forall(head: T, tail: List[T]) {
        if f(tail) {
            if tail.contains(head) {
                List.cons(head, tail).unique = tail.unique
                List.cons(head, tail).length = tail.length.suc
                tail.unique.length <= tail.length
                tail.unique.length < tail.length.suc
                List.cons(head, tail).unique.length <= List.cons(head, tail).length
            } else {
                List.cons(head, tail).unique.length <= List.cons(head, tail).length
            }
            f(List.cons(head, tail))
        }
    }
}

theorem contains_imp_unique_contains[T](list: List[T], item: T) {
    list.contains(item) implies list.unique.contains(item)
} by {
    define p(l: List[T]) -> Bool {
        l.contains(item) implies l.unique.contains(item)
    }

    p(List.nil[T])

    forall(head: T, tail: List[T]) {
        if p(tail) {
            if List.cons(head, tail).contains(item) {
                if tail.contains(head) {
                    List.cons(head, tail).unique.contains(item)
                } else {
                    List.cons(head, tail).unique.contains(item)
                }
                List.cons(head, tail).unique.contains(item)
            }
            p(List.cons(head, tail))
        }
    }

    p(list)
}

theorem unique_contains_imp_contains[T](list: List[T], item: T) {
    list.unique.contains(item) implies list.contains(item)
} by {
    define p(l: List[T]) -> Bool {
        l.unique.contains(item) implies l.contains(item)
    }

    p(List.nil[T])

    forall(head: T, tail: List[T]) {
        if p(tail) {
            if List.cons(head, tail).unique.contains(item) {
                if tail.contains(head) {
                    List.cons(head, tail).contains(item)
                } else {
                    List.cons(head, tail).contains(item)
                }
                List.cons(head, tail).contains(item)
            }
            p(List.cons(head, tail))
        }
    }

    p(list)
}

theorem unique_preserves_contains[T](list: List[T], item: T) {
    list.unique.contains(item) = list.contains(item)
} by {
    if list.unique.contains(item) != list.contains(item) {
        if list.unique.contains(item) {
            false
        } else {
            false
        }
    }
}

theorem unique_idemp[T](list: List[T]) {
    list.unique.unique = list.unique
} by {
    let list_unique = list.unique

    define f(x: List[T]) -> Bool {
        x.unique.unique = x.unique
    }

    // Base case
    f(List.nil)

    // Induction
    forall(head: T, tail: List[T]) {
        if f(tail) {

            if tail.contains(head) {
                // Definition of unique and induction assumption
                List.cons(head, tail).unique = tail.unique
            } else {
                // Head is not in tail, so added to end (definition)
                List.cons(head, tail).unique.unique = List.cons(head, tail).unique
            }

            f(List.cons(head, tail))
        }
    }
}

theorem unique_list_is_unique[T](list: List[T]) {
    list.unique.is_unique
}

theorem unique_implies_tail_unique[T](head: T, tail: List[T]) {
    List.cons(head, tail).is_unique implies tail.is_unique
} by {
    if tail.contains(head) {
        tail.length < List.cons(head, tail).length

        not List.cons(head, tail).is_unique
    } else {
    }
}

attributes List[T] {
    /// The number of times the given item appears in the list.
    define count(self, item: T) -> Nat {
        match self {
            List.nil[T] {
                Nat.0
            }
            List.cons(head, tail) {
                if head = item {
                    1 + tail.count(item)
                } else {
                    tail.count(item)
                }
            }
        }
    }
}

theorem list_contains_implies_count_geq_one[T](list: List[T], item: T) {
    list.contains(item) implies list.count(item) >= Nat.1
} by {
    define p(l: List[T], x: T) -> Bool {
        l.contains(x) implies l.count(x) >= Nat.1
    }

    // Base case: empty list
    p(List.nil[T], item)

    // Induction
    forall(head: T, tail: List[T]) {
        if p(tail, item) {
            if List.cons(head, tail).contains(item) {
                if head = item {
                    List.cons(head, tail).count(item) = 1 + tail.count(item)
                    List.cons(head, tail).count(item) >= Nat.1
                } else {
                    tail.contains(item)
                    List.cons(head, tail).count(item) = tail.count(item)
                    List.cons(head, tail).count(item) >= Nat.1
                }
                p(List.cons(head, tail), item)
            }
            p(List.cons(head, tail), item)
        }
    }

    List.induction(function (l: List[T]) {
        p(l, item)
    })

    forall(l: List[T]) {
        p(l, item)
    }
}

theorem list_not_contains_impl_count_zero[T](list: List[T], item: T) {
    not list.contains(item) implies list.count(item) = Nat.0
} by {
    define p(l: List[T], x: T) -> Bool {
        not l.contains(x) implies l.count(x) = Nat.0
    }

    // Base case: empty list
    List.nil[T].count(item) = Nat.0
    p(List.nil[T], item)

    // Induction
    forall(head: T, tail: List[T]) {
        if p(tail, item) {
            if List.cons(head, tail).contains(item) {
                if head = item {
                    List.cons(head, tail).count(item) >= Nat.1
                } else {
                    tail.contains(item)
                    List.cons(head, tail).count(item) >= Nat.1
                }
            }
            if not List.cons(head, tail).contains(item) {
                // head != item and item not in tail
                not tail.contains(item)

                // By induction hypothesis

                // Since head != item, count stays the same
                List.cons(head, tail).count(item) = tail.count(item)
                List.cons(head, tail).count(item) = Nat.0
            }
            // In either case, contains(item) implies count >= 1
            p(List.cons(head, tail), item)
        }
    }

    List.induction(function (l: List[T]) {
        p(l, item)
    })

    forall(l: List[T]) {
        p(l, item)
    }

}

theorem unique_implies_no_duplicate[T](list: List[T], item: T) {
    list.is_unique implies list.count(item) <= Nat.1
} by {
    define p(l: List[T], x: T) -> Bool {
        l.is_unique implies l.count(x) <= Nat.1
    }

    // Base case: empty list
    List.nil[T].count(item) = Nat.0
    Nat.0 <= Nat.1
    p(List.nil[T], item)

    // Induction
    forall(head: T, tail: List[T]) {
        if p(tail, item) {
            if List.cons(head, tail).is_unique {
                // tail is also unique
                tail.is_unique

                if head = item {
                    // Since cons is unique, head (= item) is not in tail
                    not tail.contains(item)
                    tail.count(item) = 0
                    List.cons(head, tail).count(item) = 1 + tail.count(item)
                    List.cons(head, tail).count(item) = 1
                    List.cons(head, tail).count(item) <= 1
                } else {
                    List.cons(head, tail).count(item) <= 1
                }
                p(List.cons(head, tail), item)
            }
            p(List.cons(head, tail), item)
        }
    }

    List.induction(function (l: List[T]) {
        p(l, item)
    })
    forall(l: List[T]) {
        p(l, item)
    }
}

theorem unique_len_smaller_implies_duplicate[T](list: List[T]) {
    list.unique.length < list.length implies exists(x: T) {
        list.count(x) > 1
    }
} by {
    define p(l: List[T]) -> Bool {
        l.unique.length < l.length implies exists(x: T) {
            l.count(x) > 1
        }
    }

    // Base case: empty list
    p(List.nil[T])

    // Induction
    forall(head: T, tail: List[T]) {
        if p(tail) {
            if List.cons(head, tail).unique.length < List.cons(head, tail).length {
                if tail.unique.length < tail.length {
                    // x is in tail, use hypothesis
                    let x: T satisfy {
                        tail.count(x) > 1
                    }

                    if head = x {
                        List.cons(head, tail).count(x) = Nat.1 + tail.count(x)
                        tail.count(x) <= tail.count(x) + Nat.1
                        List.cons(head, tail).count(x) >= tail.count(x)
                    } else {
                    }
                    List.cons(head, tail).count(x) >= tail.count(x)

                    p(List.cons(head, tail))
                } else {
                    // x is not in tail so head must be the duplicate
                    // tail is unique, but cons(head, tail) isn't, so head is in tail
                    tail.contains(head)
                    tail.count(head) >= 1

                    // cons adds 1 more to the count
                    List.cons(head, tail).count(head) = Nat.1 + tail.count(head)

                    List.cons(head, tail).count(head) > 1

                }

                p(List.cons(head, tail))
            }
            p(List.cons(head, tail))

        }
    }

    forall(l: List[T]) {
        p(l)
    }
}

theorem cons_duplicate_inequality[T](tail: List[T], head: T, x: T, n: Nat) {
    tail.count(x) >= n implies List.cons(head, tail).count(x) >= n
} by {
    if tail.count(x) >= n {
        if head = x {
            List.cons(head, tail).count(x) = 1 + tail.count(x)
            1 + tail.count(x) = tail.count(x).suc
            tail.count(x) < tail.count(x).suc
            n <= tail.count(x)
            n < tail.count(x).suc
            n <= tail.count(x).suc
            List.cons(head, tail).count(x) >= n
        } else {
            List.cons(head, tail).count(x) = tail.count(x)
            List.cons(head, tail).count(x) >= n
        }
    }
}

theorem not_unique_implies_duplicate[T](list: List[T]) {
    not list.is_unique implies exists(x: T) {
        list.count(x) > 1
    }
} by {
    define p(l: List[T]) -> Bool {
        not l.is_unique implies exists(x: T) {
            l.count(x) > 1
        }
    }

    // Base case: empty list
    p(List.nil[T])

    // Induction
    forall(head: T, tail: List[T]) {
        if p(tail) {
            if not List.cons(head, tail).is_unique {
                if not tail.is_unique {
                    let x: T satisfy {
                        tail.count(x) > 1
                    }
                    // tail.count(x) >= 2
                    tail.count(x) >= Nat.1.suc

                    if x = head {
                        // cons adds 1 more
                        List.cons(head, tail).count(x) >= Nat.1.suc
                    } else {
                        // x != head, so cons adds nothing but tail already has >= 2
                        List.cons(head, tail).count(x) >= Nat.1.suc
                    }
                    List.cons(head, tail).count(x) > 1

                } else {
                    // If tail is unique, then head must be the duplicate
                    tail.is_unique
                    tail.contains(head)
                    tail.count(head) >= Nat.1
                    tail.count(head) <= Nat.1
                    tail.count(head) = 1
                    List.cons(head, tail).count(head) = 1 + tail.count(head)
                    1 < Nat.1.suc
                    List.cons(head, tail).count(head) > 1

                }
                p(List.cons(head, tail))
            }

        }
    }
}

attributes List[T] {
    /// Appends a single element to the end of the list.
    define append(self, item: T) -> List[T] {
        self + List.singleton(item)
    }
}

attributes Nat {
    /// Creates a list of natural numbers from 0 to n-1.
    define range(self) -> List[Nat] {
        match self {
            Nat.zero {
                List.nil[Nat]
            }
            Nat.suc(n) {
                n.range.append(n)
            }
        }
    }
}

attributes List[T] {
    /// Creates a list of natural numbers from 0 to n-1.
    let range: Nat -> List[Nat] = Nat.range

    /// Filters the list, keeping only elements that satisfy the given predicate.
    define filter(self, f: T -> Bool) -> List[T] {
        match self {
            List.nil {
                List.nil[T]
            }
            List.cons(head, tail) {
                if f(head) {
                    List.cons(head, tail.filter(f))
                } else {
                    tail.filter(f)
                }
            }
        }
    }

    /// Remove all instances of an element from the list.
    define remove_elem(self, elem: T) -> List[T] {
        match self {
            List.nil {
                List.nil[T]
            }
            List.cons(head, tail) {
                if head != elem {
                    List.cons(head, tail.remove_elem(elem))
                } else {
                    tail.remove_elem(elem)
                }
            }
        }
    }
}

// Filter theorems (kinda ugly, but alas)
theorem filter_only_removes_elems[T](list: List[T], f: T -> Bool) {
    list.filter(f).length <= list.length
} by {
    define p(l: List[T]) -> Bool {
        l.filter(f).length <= l.length
    }

    // Base case: empty list
    p(List.nil[T])
    forall(head: T, tail: List[T]) {
        if p(tail) {
            // Induction hypothesis: tail.filter(f).length <= tail.length
            if f(head) {
                // filter keeps head, so filter length = tail.filter(f).length + 1
                List.cons(head, tail).filter(f) = List.cons(head, tail.filter(f))
                List.cons(head, tail).filter(f).length = tail.filter(f).length.suc

                // cons length = tail.length + 1
                List.cons(head, tail).length = tail.length.suc

                List.cons(head, tail).filter(f).length <= List.cons(head, tail).length
            } else {
                List.cons(head, tail).filter(f) = tail.filter(f)
                tail.filter(f).length <= tail.length
                tail.length < tail.length.suc
                List.cons(head, tail).filter(f).length <= List.cons(head, tail).length
            }
            p(List.cons(head, tail))
        }
    }
}

theorem filter_contains_and[T](list: List[T], f: T -> Bool, item: T) {
    (list.contains(item) and f(item)) implies list.filter(f).contains(item)
} by {
    define p(l: List[T], x: T) -> Bool {
        l.contains(x) and f(x) implies l.filter(f).contains(x)
    }

    // Base case: empty list
    p(List.nil[T], item)

    // Induction
    forall(head: T, tail: List[T]) {
        if p(tail, item) {
            if tail.contains(item) {
                if f(head) {
                    List.cons(head, tail).filter(f).contains(item)
                } else {
                    List.cons(head, tail).filter(f).contains(item)
                }
            } else {
                if List.cons(head, tail).contains(item) and f(item) {
                    head = item
                    List.cons(head, tail).filter(f) = List.cons(item, tail.filter(f))
                    List.cons(head, tail).filter(f).contains(item)
                }
            }
            p(List.cons(head, tail), item)
        }
    }

    List.induction(function (l: List[T]) {
        p(l, item)
    })
    forall(l: List[T]) {
        p(l, item)
    }
}

theorem filter_contained_by_and[T](list: List[T], f: T -> Bool, item: T) {
    list.filter(f).contains(item) implies (list.contains(item) and f(item))
} by {
    define p(l: List[T], x: T) -> Bool {
        l.filter(f).contains(x) implies (l.contains(x) and f(x))
    }

    // Base case: empty list
    p(List.nil[T], item)

    // Induction
    forall(head: T, tail: List[T]) {
        if p(tail, item) {
            if List.cons(head, tail).filter(f).contains(item) {
                // If the filter contains the item then item must either
                // be in tail or head
                if tail.filter(f).contains(item) {
                    // `and` phrase here is needed to help prover
                } else {
                    // Since filter(f) contains item but tail.filter(f) doesn't,
                    // item must be head and f(head) must be true
                    f(head)
                    head = item
                    f(item) and List.cons(head, tail).contains(item)
                }
            }
            p(List.cons(head, tail), item)
        }
    }

    List.induction(function (l: List[T]) {
        p(l, item)
    })
    forall(l: List[T]) {
        p(l, item)
    }
    p(list, item)
}

theorem filter_equivalent_to_and[T](list: List[T], f: T -> Bool, item: T) {
    (list.contains(item) and f(item)) = list.filter(f).contains(item)
} by {
    list.filter(f).contains(item) implies (list.contains(item) and f(item))
}

/// Removing an element is the same as filtering for elements unequal to it.
theorem remove_elem_eq_filter[T](list: List[T], elem: T) {
    list.remove_elem(elem) = list.filter(function(x: T) { x != elem })
} by {
    define p(l: List[T]) -> Bool {
        l.remove_elem(elem) = l.filter(function(x: T) { x != elem })
    }

    p(List.nil[T])

    forall(head: T, tail: List[T]) {
        if p(tail) {
            if head != elem {
                List.cons(head, tail).remove_elem(elem) = List.cons(head, tail.remove_elem(elem))
                List.cons(head, tail).filter(function(x: T) { x != elem }) =
                    List.cons(head, tail.filter(function(x: T) { x != elem }))
                tail.remove_elem(elem) = tail.filter(function(x: T) { x != elem })
                List.cons(head, tail).remove_elem(elem) =
                    List.cons(head, tail).filter(function(x: T) { x != elem })
            } else {
                List.cons(head, tail).remove_elem(elem) = tail.remove_elem(elem)
                List.cons(head, tail).filter(function(x: T) { x != elem }) =
                    tail.filter(function(x: T) { x != elem })
                tail.remove_elem(elem) = tail.filter(function(x: T) { x != elem })
                List.cons(head, tail).remove_elem(elem) =
                    List.cons(head, tail).filter(function(x: T) { x != elem })
            }
            p(List.cons(head, tail))
        }
    }

    List.induction(function(l: List[T]) {
        p(l)
    })
    forall(l: List[T]) {
        p(l)
    }
}

/// Removing an element never increases the length of a list.
theorem remove_elem_length_le[T](list: List[T], elem: T) {
    list.remove_elem(elem).length <= list.length
} by {
    remove_elem_eq_filter(list, elem)
    filter_only_removes_elems(list, function(x: T) { x != elem })
}

/// Removing an element preserves exactly the other elements of the list.
theorem remove_elem_contains_iff[T](list: List[T], elem: T, item: T) {
    list.remove_elem(elem).contains(item) = (list.contains(item) and item != elem)
} by {
    remove_elem_eq_filter(list, elem)
    filter_equivalent_to_and(list, function(x: T) { x != elem }, item)
}

/// Removing an element leaves no occurrence of that element in the list.
theorem remove_elem_not_contains_elem[T](list: List[T], elem: T) {
    not list.remove_elem(elem).contains(elem)
} by {
    remove_elem_contains_iff(list, elem, elem)
    if list.remove_elem(elem).contains(elem) {
        false
    }
}

theorem filter_of_super_is_self[T](list: List[T], filter: T -> Bool) {
    forall(x: T) {
        list.contains(x) implies filter(x)
    } implies list.filter(filter) = list
} by {
    // Induction
    define f(x: List[T]) -> Bool {
        forall(y: T) {
            x.contains(y) implies filter(y)
        } implies x.filter(filter) = x
    }

    // Base case: empty list
    f(List.nil)

    // Inductive step
    forall(head: T, tail: List[T]) {
            if f(tail) {
            if forall(y: T) { List.cons(head, tail).contains(y) implies filter(y) } {
                filter(head)
                forall(y: T) {
                    if y ∈ tail {
                        y ∈ List.cons(head, tail)
                        filter(y)
                    }
                }
                forall(y: T) { tail.contains(y) implies filter(y) }
                tail.filter(filter) = tail
                List.cons(head, tail).filter(filter) = List.cons(head, tail.filter(filter))
                List.cons(head, tail).filter(filter) = List.cons(head, tail)
            }
            f(List.cons(head, tail))
        }
    }
    f(list)
}

theorem filter_of_self_is_self[T](list: List[T]) {
    list.filter(list.contains) = list
}

// Length theorems
theorem length_range(n: Nat) {
    n.range.length = n
} by {
    // Induction on n
    define f(x: Nat) -> Bool {
        x.range.length = x
    }

    // Base case: 0.range.length = 0
    f(0)

    // Inductive step
    forall(x: Nat) {
        if f(x) {
            // Induction hypothesis: x.range.length = x

            // Use add_length theorem

            // Simplify the left side
            x.range.length + List.singleton(x).length = x.suc

            // Therefore
            f(x.suc)
        }
    }

    // Conclusion
}

theorem suc_range_contains(n: Nat) {
    n.suc.range.contains(n)
} by {
    // n is in its own singleton
    List.singleton(n).contains(n)

    // n.suc.range is n.range with n appended
    n.suc.range = n.range.append(n)
}

theorem range_contains_all_leq(n: Nat) {
    forall(x: Nat) {
        x < n implies n.range.contains(x)
    }
} by {
    define p(m: Nat) -> Bool {
        forall(x: Nat) {
            x < m implies m.range.contains(x)
        }
    }

    // Base case: 0
    p(Nat.0)

    // Induction
    forall(m: Nat) {
        if p(m) {
            // Induction hypothesis: forall(x: Nat) { x < m implies m.range.contains(x) }

            // m.suc.range = m.range.append(m)
            m.suc.range = m.range.append(m)

            forall(x: Nat) {
                if x < m.suc {
                    if x < m {
                        // By induction hypothesis, x is in m.range
                        m.range.contains(x)

                        m.suc.range.contains(x)
                    } else {
                        m.suc.range.contains(x)
                    }
                    m.suc.range.contains(x)
                }
            }
            p(m.suc)
        }
    }

    forall(k: Nat) { p(k) }

    // Conclusion
}

theorem range_does_not_contain_geq(m: Nat, n: Nat) {
    n >= m implies not m.range.contains(n)
} by {
    define p(x: Nat) -> Bool {
        forall(y: Nat) {
            y >= x implies not x.range.contains(y)
        }
    }

    // Base case: 0
    p(0)

    // Induction
    forall(x: Nat) {
        if p(x) {
            forall(y: Nat) {
                if y >= x.suc {
                    not x.range.contains(y)
                    y != x

                    // x.suc.range = x.range.append(x) = x.range + singleton(x)
                    x.range.append(x) = x.range + List.singleton(x)

                    // y not in the concatenation
                    not (x.range + List.singleton(x)).contains(y)

                    not x.suc.range.contains(y)
                }
            }
            p(x.suc)
        }
    }
}

/// Any member of `n.range` is strictly below `n`.
theorem lt_of_range_contains(n: Nat, x: Nat) {
    n.range.contains(x) implies x < n
} by {
    define p(m: Nat) -> Bool {
        forall(y: Nat) {
            m.range.contains(y) implies y < m
        }
    }

    p(Nat.0)

    forall(m: Nat) {
        if p(m) {

            forall(y: Nat) {
                if m.suc.range.contains(y) {
                    add_contains_or(m.range, List.singleton(m), y)
                    m.range.contains(y) or List.singleton(m).contains(y)
                    if m.range.contains(y) {
                        y < m
                        y < m.suc
                    } else {
                        singleton_contains_imp_eq[Nat](m, y)
                        y < m.suc
                    }
                    y < m.suc
                }
            }
            p(m.suc)
        }
    }

    forall(m: Nat) { p(m) }
    if n.range.contains(x) {
        x < n
    }
}

/// Every natural number strictly below `n` belongs to `n.range`.
theorem range_contains_of_lt(n: Nat, x: Nat) {
    x < n implies n.range.contains(x)
} by {
    if x < n {
        range_contains_all_leq(n)
        n.range.contains(x)
    }
}

/// Membership in `n.range` is exactly strict boundedness by `n`.
theorem range_contains_iff_lt(n: Nat, x: Nat) {
    n.range.contains(x) = (x < n)
} by {
    if n.range.contains(x) {
        lt_of_range_contains(n, x)
        x < n
    }
    if x < n {
        range_contains_of_lt(n, x)
        n.range.contains(x)
    }
}

// If a list does not contain an element, removing it does not change the list.
theorem remove_element_not_in_list[T](list: List[T], item: T) {
    not list.contains(item) implies list.remove_elem(item) = list
} by {
    define p(l: List[T], x: T) -> Bool {
        not l.contains(x) implies l.remove_elem(x) = l
    }

    // Base case: empty list
    List.nil[T].remove_elem(item) = List.nil[T]
    p(List.nil[T], item)

    // Induction
    forall(head: T, tail: List[T]) {
        if p(tail, item) {
            if not List.cons(head, tail).contains(item) {
                // head != item and item not in tail
                not tail.contains(item)

                // By induction hypothesis
                tail.remove_elem(item) = tail

                // Since head != item, remove_elem distributes
                List.cons(head, tail).remove_elem(item) = List.cons(head, tail.remove_elem(item))

                List.cons(head, tail).remove_elem(item) = List.cons(head, tail)
            }
            p(List.cons(head, tail), item)
        }
    }

    List.induction(function (l: List[T]) {
        p(l, item)
    })
}

/// Removing the same element twice is the same as removing it once.
theorem remove_elem_idempotent[T](list: List[T], elem: T) {
    list.remove_elem(elem).remove_elem(elem) = list.remove_elem(elem)
} by {
    remove_elem_not_contains_elem(list, elem)
    remove_element_not_in_list(list.remove_elem(elem), elem)
}

// Calling `remove_element(item)` on a list containing `item` actually does
// remove all instances of the element.
theorem remove_element_removes_element[T](list: List[T], item: T) {
    list.contains(item) implies not list.remove_elem(item).contains(item)
} by {
    define p(l: List[T], x: T) -> Bool {
        l.contains(x) implies not l.remove_elem(x).contains(x)
    }

    // Base case: empty list
    not List.nil[T].contains(item)
    p(List.nil[T], item)

    // Induction
    forall(head: T, tail: List[T]) {
        if p(tail, item) {
            let cons_list = List.cons(head, tail)
            if cons_list.contains(item) {
                // If the cons list contains the item, then it must be in either head or tail
                if head = item {
                    if tail.contains(item) {
                        not tail.remove_elem(item).contains(item)
                    } else {
                        tail.remove_elem(item) = tail
                    }
                    not cons_list.remove_elem(item).contains(item)
                } else {
                    not cons_list.remove_elem(item).contains(item)
                }
                not cons_list.remove_elem(item).contains(item)

            }
            p(List.cons(head, tail), item)
        }
    }

    List.induction(function (l: List[T]) {
        p(l, item)
    })
}

// Removing an element from a list does not affect the presence of other elements.
theorem remove_element_removes_no_other_elements[T](list: List[T], removed_item: T, elem: T) {
    removed_item != elem implies list.remove_elem(removed_item).contains(elem) = list.contains(elem)
} by {
    define p(l: List[T], r: T, e: T) -> Bool {
        r != e implies l.remove_elem(r).contains(e) = l.contains(e)
    }

    // Base case: empty list
    List.nil[T].remove_elem(removed_item) = List.nil[T]
    p(List.nil[T], removed_item, elem)

    // Induction
    forall(head: T, tail: List[T]) {
        let cons_list = List.cons(head, tail)
        if p(tail, removed_item, elem) {
            // Hypothesis
            if removed_item != elem {
                // If we don't remove anything, then trivial

                // If the removed item is contained, it is either in head or tail
                // If head, then cons_list.remove_elem(removed_item) = tail.remove_elem(removed_item)
                if head = removed_item {

                } else {
                    // If the removed item is in tail then induction takes care of the rest
                    cons_list.remove_elem(removed_item) = List.cons(head, tail.remove_elem(removed_item))
                    cons_list.remove_elem(removed_item).contains(elem) = (head = elem) or tail.remove_elem(removed_item).contains(elem)

                    if head = elem {

                    } else {
                        cons_list.remove_elem(removed_item).contains(elem) = tail.remove_elem(removed_item).contains(elem)
                        tail.remove_elem(removed_item).contains(elem) = tail.contains(elem)
                        cons_list.contains(elem) = tail.contains(elem)
                        p(cons_list, removed_item, elem)
                    }

                    p(cons_list, removed_item, elem)
                }
            }
            p(List.cons(head, tail), removed_item, elem)
        }
    }

    List.induction(function (l: List[T]) {
        p(l, removed_item, elem)
    })

}

// Removing an element contained in a list results in a `.unique` list with one fewer element.
// Statement is list.unique.remove_elem(item).length + Nat.1 = list.unique.length rather than
// list.unique.remove_elem(item).length = list.unique.length - Nat.1 because cancellation laws
// don't work if list.unique.length = Nat.0.
theorem remove_element_in_unique_equals_length_minus_one[T](list: List[T], item: T) {
    list.contains(item) implies list.unique.remove_elem(item).length + Nat.1 = list.unique.length
} by {
    define p(l: List[T]) -> Bool {
        l.contains(item) implies l.unique.remove_elem(item).length + Nat.1 = l.unique.length
    }

    // Base case: empty list, does not contain any item
    p(List.nil)

    // Induction
    forall(head: T, tail: List[T]) {
        if p(tail) {
            if List.cons(head, tail).contains(item) {
                // Either head is a duplicate or not
                if tail.contains(head) {
                    // If it's a duplicate, we can safely ignore it and the induction hypothesis
                    // carries us through
                    List.cons(head, tail).unique = tail.unique
                    if head = item {
                        tail.contains(item)
                    } else {
                        tail.contains(item)
                    }
                    tail.unique.remove_elem(item).length + Nat.1 = tail.unique.length
                    List.cons(head, tail).unique.remove_elem(item).length + Nat.1 = List.cons(head, tail).unique.length
                } else {
                    // If it is not a duplicate, then either the item is head or in tail.unique
                    if head = item {
                        // If the item is in head, it's not in tail.unique, and it will be one removed

                        // Prover help: item is head, remove
                        List.cons(head, tail).unique.remove_elem(item) = tail.unique

                        List.cons(head, tail).unique.remove_elem(item).length + Nat.1 = List.cons(head, tail).unique.length
                    } else {
                        // If it's in tail then we're done by induction hypothesis

                        // Since head != item and cons contains item, item must be in tail
                        tail.contains(item)

                        // By induction hypothesis
                        tail.unique.remove_elem(item).length + Nat.1 = tail.unique.length

                        // Since head is not in tail, unique is cons
                        List.cons(head, tail).unique = List.cons(head, tail.unique)

                        // Since head != item, remove_elem distributes
                        List.cons(head, tail.unique).remove_elem(item) = List.cons(head, tail.unique.remove_elem(item))

                        List.cons(head, tail).unique.remove_elem(item).length + Nat.1 = List.cons(head, tail).unique.length
                    }
                }
            }
            p(List.cons(head, tail))
        }
    }
    p(list)
}

theorem unique_comm_remove_elem[T](list: List[T], elem: T) {
    list.remove_elem(elem).unique = list.unique.remove_elem(elem)
} by {
    define p(l: List[T], e: T) -> Bool {
        l.remove_elem(e).unique = l.unique.remove_elem(e)
    }

    // Base case: empty list
    p(List.nil[T], elem)

    forall(head: T, tail: List[T]) {
        if p(tail, elem) {
            if head = elem {
                // If elem is head, easy
                List.cons(head, tail).remove_elem(elem) = tail.remove_elem(elem)
                tail.remove_elem(elem).unique = tail.unique.remove_elem(elem)
                if tail.contains(head) {
                    List.cons(head, tail).unique = tail.unique
                } else {
                    List.cons(head, tail.unique).remove_elem(elem) = tail.unique.remove_elem(elem)
                }
                List.cons(head, tail).unique.remove_elem(elem) = tail.unique.remove_elem(elem)
                p(List.cons(head, tail), elem)
            } else {
                // If not, then either elem is in tail or not
                // Now, either head is in tail or not
                if tail.contains(head) {
                    // If head is in tail, then tail.unique carries us through
                } else {
                    // Otherwise, head is not in tail and elem is not head so unique and remove contains head
                    p(List.cons(head, tail), elem)
                }

            }

            p(List.cons(head, tail), elem)
        }
    }

    List.induction(function (l: List[T]) {
        p(l, elem)
    })

}

// And the big one: the smallest (by `.length`) list containing all elements of
// a list is its `.unique`. This now enables us to talk about "cardinality" of a
// set as being unique via `.filter(set.contains).unique` or, by the previous
// theorem, via `.unique.filter(set.contains)`.
theorem unique_is_smallest_containing_list[T](list: List[T], container: List[T]) {
    forall(x: T) {
        list.contains(x) implies container.contains(x)
    } implies list.unique.length <= container.length
} by {
    // Simple helper functions here
    define is_contained(a: T -> Bool, b: T -> Bool) -> Bool {
        forall(x: T) {
            a(x) implies b(x)
        }
    }

    define pc(x: List[T], c: List[T]) -> Bool {
        is_contained(x.contains, c.contains) implies x.unique.length <= c.unique.length
    }

    define p(x: List[T]) -> Bool {
        forall(c: List[T]) {
            pc(x, c)
        }
    }

    forall(c: List[T]) {
        pc(List.nil[T], c)
    }
    p(List.nil[T])

    forall(head: T, tail: List[T]) {
        if p(tail) {
            forall(c: List[T]) {
                if is_contained(List.cons(head, tail).contains, c.contains) {
                    // Either head is in tail or not
                    if tail.contains(head) {

                        // Hypothesis
                        forall(x: T) {
                            if tail.contains(x) {
                                List.cons(head, tail).contains(x)
                                c.contains(x)
                            }
                        }
                        is_contained(tail.contains, c.contains)

                    } else {
                        // Head is not in the tail, so gets added

                        // Remove one from the container

                        // Prove that tail.contains(x) implies c.remove_elem(head).contains(x)
                        forall(x: T) {
                            if x != head {
                                tail.contains(x) implies c.remove_elem(head).contains(x)
                            } else {
                            }

                        }
                        forall(x: T) {
                            tail.contains(x) implies c.remove_elem(head).contains(x)
                        }
                        is_contained(tail.contains, c.remove_elem(head).contains)

                        // Given this, we know know that adding head to both should give result
                        List.cons(head, tail).contains(head)
                        c.contains(head)
                        c.unique.remove_elem(head) = c.remove_elem(head).unique
                        c.unique.remove_elem(head).length + Nat.1 = c.unique.length
                        c.remove_elem(head).unique.length + Nat.1 = c.unique.length

                        tail.unique.length + Nat.1 <= c.remove_elem(head).unique.length + Nat.1
                        List.cons(head, tail).unique.length <= c.unique.length
                    }

                    pc(List.cons(head, tail), c)
                }
            }
            p(List.cons(head, tail))
        }
    }

    p(list)
    pc(list, container)
    container.unique.length <= container.length
    if is_contained(list.contains, container.contains) {
        list.unique.length <= container.unique.length
        list.unique.length <= container.length
    }
}

/// The sum of two unique `List`s with no common elements is unique.
theorem unique_list_sum[T](list1: List[T], list2: List[T]) {
    list1.is_unique and list2.is_unique and forall(x: T) {
        not (list1.contains(x) and list2.contains(x))
    } implies (list1 + list2).is_unique
} by {
    define disjoint(l1: List[T], l2: List[T]) -> Bool {
        forall(x: T) {
            not (l1.contains(x) and l2.contains(x))
        }
    }

    define p(l1: List[T], l2: List[T]) -> Bool {
        l1.is_unique and l2.is_unique and disjoint(l1, l2) implies (l1 + l2).is_unique
    }

    // Base case: empty list1
    p(List.nil[T], list2)
    forall(head: T, tail: List[T]) {
        if p(tail, list2) {
            if List.cons(head, tail).is_unique {
                if list2.is_unique {
                    if disjoint(List.cons(head, tail), list2) {

                        // Induction hypothesis
                        tail.is_unique
                        forall(x: T) {
                            tail.contains(x) implies List.cons(head, tail).contains(x)
                            not (tail.contains(x) and list2.contains(x))
                        }
                        disjoint(tail, list2)
                        (tail + list2).is_unique

                        // Since tail + list2 is unique, its unique is itself
                        (tail + list2).unique = tail + list2

                        // head is not in tail (from cons being unique)
                        not tail.contains(head)

                        // head is not in list2 (from disjoint)
                        List.cons(head, tail).contains(head)
                        not list2.contains(head)

                        // Therefore head is not in tail + list2
                        not (tail + list2).contains(head)

                        // So cons(head, tail + list2) is also unique

                        // Now, adding head to the front (this is the hard part of the proof)
                        List.cons(head, tail + list2).unique = List.cons(head, (tail + list2).unique)

                        (List.cons(head, tail) + list2).is_unique
                    }
                }
            }
            p(List.cons(head, tail), list2)
        }

    }

    List.induction(function (l1: List[T]) {
        p(l1, list2)
    })
    forall(l1: List[T]) {
        p(l1, list2)
    }

}

/// The sum of two unique `List`s with no common elements is the sum of their lengths.
theorem unique_list_sum_length[T](list1: List[T], list2: List[T]) {
    list1.is_unique and list2.is_unique and forall(x: T) {
        not (list1.contains(x) and list2.contains(x))
    } implies (list1 + list2).unique.length = list1.length + list2.length
}

/// Inductive step for `range_is_unique`: appending a fresh maximum preserves
/// uniqueness, since the new element is not yet present.
theorem range_is_unique_suc(x: Nat) {
    x.range.is_unique implies x.suc.range.is_unique
} by {
    if x.range.is_unique {
        singleton_unique(x)
        range_does_not_contain_geq(x, x)
        forall(y: Nat) {
            if List.singleton(x).contains(y) {
                y = x
                not x.range.contains(y)
            }
            not (x.range.contains(y) and List.singleton(x).contains(y))
        }
        unique_list_sum(x.range, List.singleton(x))
        (x.range + List.singleton(x)).is_unique
        x.range.append(x) = x.range + List.singleton(x)
        x.suc.range.is_unique
    }
}

/// Every range list `n.range` has no duplicates.
theorem range_is_unique(n: Nat) {
    n.range.is_unique
} by {
    define p(x: Nat) -> Bool {
        x.range.is_unique
    }
    List.nil[Nat].is_unique
    p(Nat.0)
    forall(x: Nat) {
        if p(x) {
            range_is_unique_suc(x)
            p(x.suc)
        }
    }
    forall(x: Nat) { p(x) implies p(x.suc) }
    p(Nat.0) and forall(x: Nat) { p(x) implies p(x.suc) }
    alt_induction(p)
    forall(x: Nat) { p(x) }
}

/// Applies a function to each element of a list, creating a new list of results.
attributes List[T] {
    define map[U](self, f: T -> U) -> List[U] {
        match self {
            List.nil {
                List.nil[U]
            }
            List.cons(head, tail) {
                List.cons(f(head), tail.map(f))
            }
        }
    }
}

/// Applies a function to each element of a list, creating a new list of results.
/// Deprecated non-attribute definition.
define map[T, U](items: List[T], f: T -> U) -> List[U] {
    match items {
        List.nil {
            List.nil[U]
        }
        List.cons(head, tail) {
            List.cons(f(head), map(tail, f))
        }
    }
}

// The two definitions of map are the same
theorem map_equivalence[T, U] {
    List.map[T, U] = map[T, U]
} by {
    forall(f: T -> U) {
        // Induction on p
        define p(ts: List[T]) -> Bool {
            ts.map(f) = map(ts, f)
        }

        // Base case
        map[T, U](List.nil[T], f) = List.nil[U]
        List.nil[T].map(f) = List.nil[U]
        p(List.nil)

        // Inductive step
        forall(head: T, tail: List[T]) {
            if p(tail) {
                map[T, U](tail, f) = tail.map(f)
                p(List.cons(head, tail))
            }
        }

        forall(ts: List[T]) {
            ts.map(f) = map(ts, f)
        }
    }
}

/// Mapping the identity function over a list leaves the list unchanged.
theorem map_identity[T](items: List[T]) {
    map[T, T](items, identity_fn[T]) = items
} by {
    define p(xs: List[T]) -> Bool {
        map[T, T](xs, identity_fn[T]) = xs
    }
    map[T, T](List.nil[T], identity_fn[T]) = List.nil[T]
    p(List.nil[T])
    forall(head: T, tail: List[T]) {
        if p(tail) {
            map[T, T](tail, identity_fn[T]) = tail
            p(List.cons(head, tail))
        }
    }
    List.induction(function(xs: List[T]) { p(xs) })
    forall(xs: List[T]) { p(xs) }
}
