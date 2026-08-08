from list.list_base import List, length_range, list_contains_implies_count_geq_one,
    list_not_contains_impl_count_zero, map, range_contains_iff_lt,
    range_contains_of_lt, range_is_unique, unique_implies_no_duplicate,
    not_unique_implies_duplicate, unique_implies_tail_unique, unique_length,
    unique_is_smallest_containing_list
from list.list_product import product
from list.list_sum import map_contains, map_length, sum
from algebra.add_comm_monoid import AddCommMonoid
from algebra.comm_monoid import CommMonoid
from nat import Nat, add_one_left, alt_suc_ne_zero, lte_imp_not_lt, lte_trans,
    not_lt_zero, only_zero_lte_zero
numerals Nat

attributes List[T] {
    /// The list with the first occurrence of the given element removed.
    /// If the element is not present, the list is unchanged.
    define remove_one(self, item: T) -> List[T] {
        match self {
            List.nil {
                List.nil[T]
            }
            List.cons(head, tail) {
                if head = item {
                    tail
                } else {
                    List.cons(head, tail.remove_one(item))
                }
            }
        }
    }
}

/// Cons step for `remove_one` when head matches.
theorem remove_one_cons_eq[T](head: T, tail: List[T]) {
    List.cons(head, tail).remove_one(head) = tail
}

/// Cons step for `remove_one` when head does not match.
theorem remove_one_cons_neq[T](head: T, tail: List[T], item: T) {
    head != item implies List.cons(head, tail).remove_one(item) = List.cons(head, tail.remove_one(item))
}

/// Removing an element absent from the list leaves the list unchanged.
theorem remove_one_not_contains[T](list: List[T], item: T) {
    not list.contains(item) implies list.remove_one(item) = list
} by {
    define p(l: List[T]) -> Bool {
        not l.contains(item) implies l.remove_one(item) = l
    }
    List.nil[T].remove_one(item) = List.nil[T]
    p(List.nil[T])
    forall(head: T, tail: List[T]) {
        if p(tail) {
            if not List.cons(head, tail).contains(item) {
                if head = item {
                    false
                }
                if tail.contains(item) {
                    false
                }
                tail.remove_one(item) = tail
                remove_one_cons_neq(head, tail, item)
                List.cons(head, tail).remove_one(item) = List.cons(head, tail)
            }
            p(List.cons(head, tail))
        }
    }
    List.induction(function(l: List[T]) { p(l) })
}

/// Removing one occurrence drops the count of that element by one.
theorem remove_one_count_self[T](list: List[T], item: T) {
    list.contains(item) implies list.remove_one(item).count(item) + Nat.1 = list.count(item)
} by {
    define p(l: List[T]) -> Bool {
        l.contains(item) implies l.remove_one(item).count(item) + Nat.1 = l.count(item)
    }
    p(List.nil[T])
    forall(head: T, tail: List[T]) {
        if p(tail) {
            if List.cons(head, tail).contains(item) {
                if head = item {
                    remove_one_cons_eq(head, tail)
                    List.cons(head, tail).count(item) = Nat.1 + tail.count(item)
                    List.cons(head, tail).remove_one(item).count(item) + Nat.1 = List.cons(head, tail).count(item)
                } else {
                    if tail.contains(item) {
                    } else {
                        false
                    }
                    tail.remove_one(item).count(item) + Nat.1 = tail.count(item)
                    remove_one_cons_neq(head, tail, item)
                    List.cons(head, tail.remove_one(item)).count(item) = tail.remove_one(item).count(item)
                    List.cons(head, tail).count(item) = tail.count(item)
                    List.cons(head, tail).remove_one(item).count(item) + Nat.1 = List.cons(head, tail).count(item)
                }
            }
            if not List.cons(head, tail).contains(item) {
            }
            p(List.cons(head, tail))
        }
    }
    List.induction(function(l: List[T]) { p(l) })
}

/// Removing one occurrence of an item leaves counts of other items unchanged.
theorem remove_one_count_other[T](list: List[T], item: T, other: T) {
    item != other implies list.remove_one(item).count(other) = list.count(other)
} by {
    define p(l: List[T]) -> Bool {
        item != other implies l.remove_one(item).count(other) = l.count(other)
    }
    p(List.nil[T])
    forall(head: T, tail: List[T]) {
        if p(tail) {
            if item != other {
                tail.remove_one(item).count(other) = tail.count(other)
                if head = item {
                    remove_one_cons_eq(head, tail)
                    List.cons(head, tail).count(other) = tail.count(other)
                    List.cons(head, tail).remove_one(item).count(other) = List.cons(head, tail).count(other)
                } else {
                    remove_one_cons_neq(head, tail, item)
                    if head = other {
                        List.cons(head, tail).count(other) = Nat.1 + tail.count(other)
                        List.cons(head, tail).remove_one(item).count(other) = Nat.1 + tail.remove_one(item).count(other)
                        List.cons(head, tail).remove_one(item).count(other) = List.cons(head, tail).count(other)
                    } else {
                        List.cons(head, tail.remove_one(item)).count(other) = tail.remove_one(item).count(other)
                        List.cons(head, tail).remove_one(item).count(other) = List.cons(head, tail).count(other)
                    }
                }
            }
            p(List.cons(head, tail))
        }
    }
    List.induction(function(l: List[T]) { p(l) })
}

/// True if two lists contain the same elements with the same multiplicities.
define is_permutation[T](a: List[T], b: List[T]) -> Bool {
    forall(x: T) { a.count(x) = b.count(x) }
}

/// Permutation is reflexive.
theorem permutation_refl[T](a: List[T]) {
    is_permutation(a, a)
}

/// Permutation is symmetric.
theorem permutation_symm[T](a: List[T], b: List[T]) {
    is_permutation(a, b) implies is_permutation(b, a)
} by {
    if is_permutation(a, b) {
        forall(x: T) {
            b.count(x) = a.count(x)
        }
    }
}

/// Permutation is transitive.
theorem permutation_trans[T](a: List[T], b: List[T], c: List[T]) {
    is_permutation(a, b) and is_permutation(b, c) implies is_permutation(a, c)
} by {
    if is_permutation(a, b) and is_permutation(b, c) {
        forall(x: T) {
            a.count(x) = b.count(x)
            b.count(x) = c.count(x)
            a.count(x) = c.count(x)
        }
    }
}

/// A list whose every-element count is zero must be empty.
theorem list_count_all_zero_imp_nil[T](list: List[T]) {
    (forall(x: T) { list.count(x) = Nat.0 }) implies list = List.nil[T]
} by {
    define p(l: List[T]) -> Bool {
        (forall(x: T) { l.count(x) = Nat.0 }) implies l = List.nil[T]
    }
    p(List.nil[T])
    forall(head: T, tail: List[T]) {
        if p(tail) {
            if forall(x: T) { List.cons(head, tail).count(x) = Nat.0 } {
                List.cons(head, tail).count(head) = Nat.1 + tail.count(head)
                Nat.1 + tail.count(head) = Nat.0
                false
            }
            p(List.cons(head, tail))
        }
    }
    List.induction(function(l: List[T]) { p(l) })
}

/// All counts in the empty list are zero.
theorem nil_count_zero[T](item: T) {
    List.nil[T].count(item) = Nat.0
}

/// A list permutation-equivalent to the empty list is itself empty.
theorem permutation_nil_imp_nil[T](list: List[T]) {
    is_permutation(list, List.nil[T]) implies list = List.nil[T]
} by {
    define p(l: List[T]) -> Bool {
        is_permutation(l, List.nil[T]) implies l = List.nil[T]
    }
    p(List.nil[T])
    forall(head: T, tail: List[T]) {
        if p(tail) {
            if is_permutation(List.cons(head, tail), List.nil[T]) {
                List.cons(head, tail).count(head) = Nat.1 + tail.count(head)
                List.cons(head, tail).count(head) = List.nil[T].count(head)
                nil_count_zero[T](head)
                Nat.1 + tail.count(head) = Nat.0
                false
            }
            p(List.cons(head, tail))
        }
    }
    List.induction(function(l: List[T]) { p(l) })
}

/// Permutations are preserved under removing one occurrence of an element.
theorem permutation_remove_one[T](a: List[T], b: List[T], item: T) {
    is_permutation(a, b) implies is_permutation(a.remove_one(item), b.remove_one(item))
} by {
    if is_permutation(a, b) {
        a.count(item) = b.count(item)
        forall(x: T) {
            if x = item {
                if a.contains(item) {
                    remove_one_count_self(a, item)
                    a.remove_one(item).count(item) + Nat.1 = a.count(item)
                    b.count(item) >= Nat.1
                    if not b.contains(item) {
                        list_not_contains_impl_count_zero[T](b, item)
                        b.count(item) = Nat.0
                        false
                    }
                    b.contains(item)
                    remove_one_count_self(b, item)
                    b.remove_one(item).count(item) + Nat.1 = b.count(item)
                    a.remove_one(item).count(item) = b.remove_one(item).count(item)
                } else {
                    list_not_contains_impl_count_zero[T](a, item)
                    a.count(item) = Nat.0
                    b.count(item) = Nat.0
                    if b.contains(item) {
                        list_contains_implies_count_geq_one[T](b, item)
                        b.count(item) >= Nat.1
                        false
                    }
                    remove_one_not_contains(a, item)
                    remove_one_not_contains(b, item)
                    a.remove_one(item).count(item) = b.remove_one(item).count(item)
                }
                a.remove_one(item).count(x) = b.remove_one(item).count(x)
            } else {
                remove_one_count_other(a, item, x)
                remove_one_count_other(b, item, x)
                a.count(x) = b.count(x)
                a.remove_one(item).count(x) = b.remove_one(item).count(x)
            }
        }
    }
}

/// Removing one occurrence from a list shortens it by one.
theorem remove_one_length_of_contains[T](list: List[T], item: T) {
    list.contains(item) implies list.remove_one(item).length.suc = list.length
} by {
    define p(l: List[T]) -> Bool {
        l.contains(item) implies l.remove_one(item).length.suc = l.length
    }
    p(List.nil[T])
    forall(head: T, tail: List[T]) {
        if p(tail) {
            if List.cons(head, tail).contains(item) {
                if head = item {
                    remove_one_cons_eq(head, tail)
                    List.cons(head, tail).remove_one(item).length.suc = List.cons(head, tail).length
                }
                if head != item {
                    tail.contains(item)
                    remove_one_cons_neq(head, tail, item)
                    List.cons(head, tail).remove_one(item).length.suc = tail.remove_one(item).length.suc.suc
                    tail.remove_one(item).length.suc.suc = tail.length.suc
                    List.cons(head, tail).remove_one(item).length.suc = List.cons(head, tail).length
                }
                List.cons(head, tail).remove_one(item).length.suc = List.cons(head, tail).length
            }
            if not List.cons(head, tail).contains(item) {
            }
            p(List.cons(head, tail))
        }
    }
    List.induction(function(l: List[T]) { p(l) })
}

/// Permutations preserve list length.
theorem permutation_preserves_length[T](a: List[T], b: List[T]) {
    is_permutation(a, b) implies a.length = b.length
} by {
    define p(l: List[T]) -> Bool {
        forall(other: List[T]) {
            is_permutation(l, other) implies l.length = other.length
        }
    }
    forall(other: List[T]) {
        if is_permutation(List.nil[T], other) {
            permutation_symm(List.nil[T], other)
            permutation_nil_imp_nil[T](other)
            List.nil[T].length = other.length
        }
    }
    p(List.nil[T])
    forall(head: T, tail: List[T]) {
        if p(tail) {
            forall(other: List[T]) {
                if is_permutation(List.cons(head, tail), other) {
                    List.cons(head, tail).count(head) = Nat.1 + tail.count(head)
                    List.cons(head, tail).count(head) = other.count(head)
                    other.count(head) = Nat.1 + tail.count(head)
                    other.count(head) >= Nat.1
                    if not other.contains(head) {
                        list_not_contains_impl_count_zero[T](other, head)
                        other.count(head) = Nat.0
                        false
                    }
                    permutation_remove_one(List.cons(head, tail), other, head)
                    remove_one_cons_eq(head, tail)
                    tail.length = other.remove_one(head).length
                    remove_one_length_of_contains(other, head)
                    List.cons(head, tail).length = other.length
                }
            }
            p(List.cons(head, tail))
        }
    }
    List.induction(function(l: List[T]) { p(l) })
}

/// Removing one occurrence of an element pulls a factor out of the product.
theorem product_remove_one[A: CommMonoid](list: List[A], item: A) {
    list.contains(item) implies item * product[A](list.remove_one(item)) = product[A](list)
} by {
    define p(l: List[A]) -> Bool {
        l.contains(item) implies item * product[A](l.remove_one(item)) = product[A](l)
    }
    p(List.nil[A])
    forall(head: A, tail: List[A]) {
        if p(tail) {
            if List.cons(head, tail).contains(item) {
                if head = item {
                    remove_one_cons_eq(head, tail)
                    item * product[A](List.cons(head, tail).remove_one(item)) = product[A](List.cons(head, tail))
                } else {
                    head != item
                    if tail.contains(item) {
                    } else {
                        false
                    }
                    tail.contains(item)
                    item * product[A](tail.remove_one(item)) = product[A](tail)
                    remove_one_cons_neq(head, tail, item)
                    List.cons(head, tail).remove_one(item) = List.cons(head, tail.remove_one(item))
                    product[A](List.cons(head, tail.remove_one(item))) = head * product[A](tail.remove_one(item))
                    item * (head * product[A](tail.remove_one(item))) = head * (item * product[A](tail.remove_one(item)))
                    item * product[A](List.cons(head, tail).remove_one(item)) = product[A](List.cons(head, tail))
                }
            }
            p(List.cons(head, tail))
        }
    }
    List.induction(function(l: List[A]) { p(l) })
}

/// Removing one source occurrence pulls its image out of a mapped product.
theorem product_map_remove_one[T, A: CommMonoid](
    list: List[T], item: T, f: T -> A
) {
    list.contains(item) implies
        f(item) * product[A](map(list.remove_one(item), f)) =
            product[A](map(list, f))
} by {
    define p(l: List[T]) -> Bool {
        l.contains(item) implies
            f(item) * product[A](map(l.remove_one(item), f)) =
                product[A](map(l, f))
    }
    p(List.nil[T])
    forall(head: T, tail: List[T]) {
        if p(tail) {
            if List.cons(head, tail).contains(item) {
                if head = item {
                    remove_one_cons_eq(head, tail)
                    f(item) * product[A](map(
                        List.cons(head, tail).remove_one(item), f)) =
                        product[A](map(List.cons(head, tail), f))
                } else {
                    if tail.contains(item) {
                    } else {
                        false
                    }
                    f(item) * product[A](map(tail.remove_one(item), f)) =
                        product[A](map(tail, f))
                    remove_one_cons_neq(head, tail, item)
                    product[A](map(
                        List.cons(head, tail.remove_one(item)), f)) =
                        f(head) * product[A](map(tail.remove_one(item), f))
                    let q = product[A](map(tail.remove_one(item), f))
                    f(item) * (f(head) * q) = f(head) * (f(item) * q)
                    product[A](map(List.cons(head, tail), f)) =
                        f(head) * product[A](map(tail, f))
                    f(item) * product[A](map(
                        List.cons(head, tail).remove_one(item), f)) =
                        product[A](map(List.cons(head, tail), f))
                }
            }
            p(List.cons(head, tail))
        }
    }
    List.induction(function(l: List[T]) { p(l) })
}

/// Permutations preserve product in any commutative monoid.
theorem permutation_preserves_product[A: CommMonoid](a: List[A], b: List[A]) {
    is_permutation(a, b) implies product[A](a) = product[A](b)
} by {
    define p(l: List[A]) -> Bool {
        forall(other: List[A]) {
            is_permutation(l, other) implies product[A](l) = product[A](other)
        }
    }
    forall(other: List[A]) {
        if is_permutation(List.nil[A], other) {
            permutation_symm(List.nil[A], other)
            permutation_nil_imp_nil[A](other)
            product[A](List.nil[A]) = product[A](other)
        }
    }
    p(List.nil[A])
    forall(head: A, tail: List[A]) {
        if p(tail) {
            forall(other: List[A]) {
                if is_permutation(List.cons(head, tail), other) {
                    List.cons(head, tail).count(head) = Nat.1 + tail.count(head)
                    List.cons(head, tail).count(head) = other.count(head)
                    other.count(head) = Nat.1 + tail.count(head)
                    other.count(head) >= Nat.1
                    if not other.contains(head) {
                        list_not_contains_impl_count_zero[A](other, head)
                        other.count(head) = Nat.0
                        false
                    }
                    permutation_remove_one(List.cons(head, tail), other, head)
                    remove_one_cons_eq(head, tail)
                    product[A](tail) = product[A](other.remove_one(head))
                    product_remove_one[A](other, head)
                    product[A](List.cons(head, tail)) = product[A](other)
                }
            }
            p(List.cons(head, tail))
        }
    }
    List.induction(function(l: List[A]) { p(l) })
}

/// Mapping a permutation into a commutative monoid preserves its product.
theorem permutation_preserves_mapped_product[T, A: CommMonoid](
    a: List[T], b: List[T], f: T -> A
) {
    is_permutation(a, b) implies
        product[A](map(a, f)) = product[A](map(b, f))
} by {
    define p(l: List[T]) -> Bool {
        forall(other: List[T]) {
            is_permutation(l, other) implies
                product[A](map(l, f)) = product[A](map(other, f))
        }
    }
    forall(other: List[T]) {
        if is_permutation(List.nil[T], other) {
            permutation_symm(List.nil[T], other)
            permutation_nil_imp_nil[T](other)
            product[A](map(List.nil[T], f)) = product[A](map(other, f))
        }
    }
    p(List.nil[T])
    forall(head: T, tail: List[T]) {
        if p(tail) {
            forall(other: List[T]) {
                if is_permutation(List.cons(head, tail), other) {
                    List.cons(head, tail).count(head) =
                        Nat.1 + tail.count(head)
                    List.cons(head, tail).count(head) = other.count(head)
                    other.count(head) >= Nat.1
                    if not other.contains(head) {
                        list_not_contains_impl_count_zero[T](other, head)
                        other.count(head) = Nat.0
                        false
                    }
                    permutation_remove_one(
                        List.cons(head, tail), other, head)
                    remove_one_cons_eq(head, tail)
                    product[A](map(tail, f)) =
                        product[A](map(other.remove_one(head), f))
                    product_map_remove_one[T, A](other, head, f)
                    f(head) * product[A](map(other.remove_one(head), f)) =
                        product[A](map(other, f))
                    product[A](map(List.cons(head, tail), f)) =
                        f(head) * product[A](map(tail, f))
                    f(head) * product[A](map(tail, f)) =
                        f(head) * product[A](map(other.remove_one(head), f))
                    product[A](map(List.cons(head, tail), f)) =
                        product[A](map(other, f))
                }
            }
            p(List.cons(head, tail))
        }
    }
    List.induction(function(l: List[T]) { p(l) })
    p(a)
}

/// Counts distribute over list concatenation.
theorem count_append[T](left: List[T], right: List[T], item: T) {
    (left + right).count(item) = left.count(item) + right.count(item)
} by {
    define p(l: List[T]) -> Bool {
        (l + right).count(item) = l.count(item) + right.count(item)
    }
    List.nil[T] + right = right
    List.nil[T].count(item) = Nat.0
    Nat.0 + right.count(item) = right.count(item)
    p(List.nil[T])
    forall(head: T, tail: List[T]) {
        if p(tail) {
            if head = item {
                List.cons(head, tail + right).count(item) = Nat.1 + (tail + right).count(item)
                (tail + right).count(item) = tail.count(item) + right.count(item)
                List.cons(head, tail).count(item) = Nat.1 + tail.count(item)
                Nat.1 + (tail.count(item) + right.count(item)) = (Nat.1 + tail.count(item)) + right.count(item)
                (List.cons(head, tail) + right).count(item) = List.cons(head, tail).count(item) + right.count(item)
            } else {
                List.cons(head, tail + right).count(item) = (tail + right).count(item)
                (tail + right).count(item) = tail.count(item) + right.count(item)
                List.cons(head, tail).count(item) = tail.count(item)
                (List.cons(head, tail) + right).count(item) = List.cons(head, tail).count(item) + right.count(item)
            }
            p(List.cons(head, tail))
        }
    }
    List.induction(function(l: List[T]) { p(l) })
    forall(l: List[T]) { p(l) }
}

/// The product distributes over list concatenation in a commutative monoid.
theorem product_append[A: CommMonoid](left: List[A], right: List[A]) {
    product[A](left + right) = product[A](left) * product[A](right)
} by {
    define p(l: List[A]) -> Bool {
        product[A](l + right) = product[A](l) * product[A](right)
    }
    List.nil[A] + right = right
    product[A](List.nil[A]) = A.1
    A.1 * product[A](right) = product[A](right)
    p(List.nil[A])
    forall(head: A, tail: List[A]) {
        if p(tail) {
            product[A](tail + right) = product[A](tail) * product[A](right)
            product[A](List.cons(head, tail + right)) = head * (product[A](tail) * product[A](right))
            p(List.cons(head, tail))
        }
    }
    List.induction(function(l: List[A]) { p(l) })
    forall(l: List[A]) { p(l) }
}

/// Removing one occurrence of an element pulls a summand out of the sum.
theorem sum_remove_one[A: AddCommMonoid](list: List[A], item: A) {
    list.contains(item) implies item + sum[A](list.remove_one(item)) = sum[A](list)
} by {
    define p(l: List[A]) -> Bool {
        l.contains(item) implies item + sum[A](l.remove_one(item)) = sum[A](l)
    }
    p(List.nil[A])
    forall(head: A, tail: List[A]) {
        if p(tail) {
            if List.cons(head, tail).contains(item) {
                if head = item {
                    remove_one_cons_eq(head, tail)
                    item + sum[A](List.cons(head, tail).remove_one(item)) = sum[A](List.cons(head, tail))
                } else {
                    head != item
                    if tail.contains(item) {
                    } else {
                        false
                    }
                    tail.contains(item)
                    item + sum[A](tail.remove_one(item)) = sum[A](tail)
                    remove_one_cons_neq(head, tail, item)
                    List.cons(head, tail).remove_one(item) = List.cons(head, tail.remove_one(item))
                    sum[A](List.cons(head, tail.remove_one(item))) = head + sum[A](tail.remove_one(item))
                    item + (head + sum[A](tail.remove_one(item))) = head + (item + sum[A](tail.remove_one(item)))
                    item + sum[A](List.cons(head, tail).remove_one(item)) = sum[A](List.cons(head, tail))
                }
            }
            p(List.cons(head, tail))
        }
    }
    List.induction(function(l: List[A]) { p(l) })
}

/// Permutations preserve sum in any additive commutative monoid.
theorem permutation_preserves_sum[A: AddCommMonoid](a: List[A], b: List[A]) {
    is_permutation(a, b) implies sum[A](a) = sum[A](b)
} by {
    define p(l: List[A]) -> Bool {
        forall(other: List[A]) {
            is_permutation(l, other) implies sum[A](l) = sum[A](other)
        }
    }
    forall(other: List[A]) {
        if is_permutation(List.nil[A], other) {
            permutation_symm(List.nil[A], other)
            permutation_nil_imp_nil[A](other)
            sum[A](List.nil[A]) = sum[A](other)
        }
    }
    p(List.nil[A])
    forall(head: A, tail: List[A]) {
        if p(tail) {
            forall(other: List[A]) {
                if is_permutation(List.cons(head, tail), other) {
                    List.cons(head, tail).count(head) = Nat.1 + tail.count(head)
                    List.cons(head, tail).count(head) = other.count(head)
                    other.count(head) = Nat.1 + tail.count(head)
                    other.count(head) >= Nat.1
                    if not other.contains(head) {
                        list_not_contains_impl_count_zero[A](other, head)
                        other.count(head) = Nat.0
                        false
                    }
                    permutation_remove_one(List.cons(head, tail), other, head)
                    remove_one_cons_eq(head, tail)
                    sum[A](tail) = sum[A](other.remove_one(head))
                    sum_remove_one[A](other, head)
                    sum[A](List.cons(head, tail)) = sum[A](other)
                }
            }
            p(List.cons(head, tail))
        }
    }
    List.induction(function(l: List[A]) { p(l) })
}

/// In a unique list, the count of any contained element is one.
theorem unique_contains_count_one[T](list: List[T], item: T) {
    list.is_unique and list.contains(item) implies list.count(item) = Nat.1
} by {
    if list.is_unique and list.contains(item) {
        list_contains_implies_count_geq_one(list, item)
        unique_implies_no_duplicate(list, item)
        list.count(item) >= Nat.1
        list.count(item) <= Nat.1
        list.count(item) = Nat.1
    }
}

/// In any list, the count of any non-contained element is zero.
theorem not_contains_count_zero[T](list: List[T], item: T) {
    not list.contains(item) implies list.count(item) = Nat.0
} by {
    if not list.contains(item) {
        list_not_contains_impl_count_zero(list, item)
    }
}

/// Two unique lists with the same membership are permutations of each other.
theorem unique_same_contains_imp_permutation[T](a: List[T], b: List[T]) {
    a.is_unique and b.is_unique and (forall(x: T) { a.contains(x) = b.contains(x) })
        implies is_permutation(a, b)
} by {
    if a.is_unique and b.is_unique and (forall(x: T) { a.contains(x) = b.contains(x) }) {
        forall(x: T) {
            if a.contains(x) {
                unique_contains_count_one(a, x)
                unique_contains_count_one(b, x)
                b.count(x) = Nat.1
                a.count(x) = b.count(x)
            } else {
                not_contains_count_zero(a, x)
                not_contains_count_zero(b, x)
                b.count(x) = Nat.0
                a.count(x) = b.count(x)
            }
        }
        is_permutation(a, b)
    }
}

lemma cons_unique_of_tail_unique_not_contains[T](head: T, tail: List[T]) {
    tail.is_unique and not tail.contains(head) implies List.cons(head, tail).is_unique
} by {
    if tail.is_unique and not tail.contains(head) {
        tail.unique = tail
        List.cons(head, tail).is_unique
    }
}

/// A unique Nat list of length `n` whose entries are below `n` contains exactly `n.range`.
theorem unique_nat_list_contained_by_range_length_contains_iff(items: List[Nat], n: Nat, x: Nat) {
    items.is_unique and items.length = n and
    (forall(y: Nat) { items.contains(y) implies y < n })
        implies items.contains(x) = (x < n)
} by {
    if items.is_unique and items.length = n and
        (forall(y: Nat) { items.contains(y) implies y < n }) {
        if items.contains(x) {
            x < n
        }
        if x < n {
            if not items.contains(x) {
                cons_unique_of_tail_unique_not_contains(x, items)
                List.cons(x, items).is_unique
                forall(y: Nat) {
                    if List.cons(x, items).contains(y) {
                        if x = y {
                            range_contains_of_lt(n, y)
                            n.range.contains(y)
                        } else {
                            range_contains_of_lt(n, y)
                            n.range.contains(y)
                        }
                        n.range.contains(y)
                    }
                }
                unique_is_smallest_containing_list(List.cons(x, items), n.range)
                List.cons(x, items).unique.length <= n.range.length
                List.cons(x, items).unique = List.cons(x, items)
                List.cons(x, items).length <= n.range.length
                List.cons(x, items).length = items.length.suc
                List.cons(x, items).length = n.suc
                length_range(n)
                n.range.length = n
                n.suc <= n
                false
            }
            items.contains(x)
        }
        items.contains(x) = (x < n)
    }
}

/// A unique Nat list of length `n` whose elements are all below `n` enumerates `n.range`.
theorem unique_nat_list_contained_by_range_length_imp_permutation(items: List[Nat], n: Nat) {
    items.is_unique and items.length = n and
    (forall(x: Nat) { items.contains(x) implies x < n })
        implies is_permutation(items, n.range)
} by {
    if items.is_unique and items.length = n and
        (forall(x: Nat) { items.contains(x) implies x < n }) {
        range_is_unique(n)
        forall(x: Nat) {
            unique_nat_list_contained_by_range_length_contains_iff(items, n, x)
            range_contains_iff_lt(n, x)
            items.contains(x) = n.range.contains(x)
        }
        unique_same_contains_imp_permutation(items, n.range)
        is_permutation(items, n.range)
    }
}

lemma unique_cons_not_contains[T](head: T, tail: List[T]) {
    List.cons(head, tail).is_unique implies not tail.contains(head)
} by {
    if List.cons(head, tail).is_unique {
        if tail.contains(head) {
            let cons_list = List.cons(head, tail)
            List.cons(head, tail).unique = List.cons(head, tail)
            tail.unique = List.cons(head, tail)
            unique_length(tail)
            List.cons(head, tail).length = tail.length.suc
            tail.length.suc <= tail.length
            false
        }
    }
}

lemma locally_injective_map_is_unique[T, U](items: List[T], f: T -> U) {
    items.is_unique and forall(x: T, y: T) {
        items.contains(x) and items.contains(y) and f(x) = f(y) implies x = y
    }
    implies map[T, U](items, f).is_unique
} by {
    define p(xs: List[T]) -> Bool {
        xs.is_unique and forall(x: T, y: T) {
            xs.contains(x) and xs.contains(y) and f(x) = f(y) implies x = y
        }
        implies map[T, U](xs, f).is_unique
    }

    map[T, U](List.nil[T], f) = List.nil[U]
    List.nil[U].unique = List.nil[U]
    List.nil[U].is_unique
    p(List.nil[T])

    forall(head: T, tail: List[T]) {
        if p(tail) {
            if List.cons[T](head, tail).is_unique and forall(x: T, y: T) {
                List.cons[T](head, tail).contains(x) and List.cons[T](head, tail).contains(y) and f(x) = f(y) implies x = y
            } {
                tail.is_unique
                forall(x: T, y: T) {
                    if tail.contains(x) and tail.contains(y) and f(x) = f(y) {
                        List.cons[T](head, tail).contains(x)
                        List.cons[T](head, tail).contains(y)
                        x = y
                    }
                }
                tail.is_unique and forall(x: T, y: T) {
                    tail.contains(x) and tail.contains(y) and f(x) = f(y) implies x = y
                }
                if map[T, U](tail, f).contains(f(head)) {
                    map_contains[T, U](tail, f, f(head))
                    let x: T satisfy {
                        tail.contains(x) and f(x) = f(head)
                    }
                    List.cons[T](head, tail).contains(x)
                    List.cons[T](head, tail).contains(head)
                    x = head
                    unique_cons_not_contains(head, tail)
                    false
                }
                cons_unique_of_tail_unique_not_contains(f(head), map[T, U](tail, f))
                List.cons[U](f(head), map[T, U](tail, f)).is_unique
                map[T, U](List.cons[T](head, tail), f).is_unique
            }
            p(List.cons[T](head, tail))
        }
    }

    List.induction(function(xs: List[T]) { p(xs) })
    p(items)
}

/// A locally injective map from `n.range` into `n.range` permutes `n.range`.
theorem map_range_locally_injective_bounded_is_permutation(n: Nat, f: Nat -> Nat) {
    (forall(x: Nat, y: Nat) { x < n and y < n and f(x) = f(y) implies x = y }) and
    (forall(x: Nat) { x < n implies f(x) < n })
        implies is_permutation(map(n.range, f), n.range)
} by {
    if (forall(x: Nat, y: Nat) { x < n and y < n and f(x) = f(y) implies x = y }) and
        (forall(x: Nat) { x < n implies f(x) < n }) {
        range_is_unique(n)
        forall(x: Nat, y: Nat) {
            if n.range.contains(x) and n.range.contains(y) and f(x) = f(y) {
                range_contains_iff_lt(n, x)
                range_contains_iff_lt(n, y)
                x = y
            }
        }
        locally_injective_map_is_unique[Nat, Nat](n.range, f)
        map[Nat, Nat](n.range, f).is_unique
        map_length[Nat, Nat](n.range, f)
        length_range(n)
        map[Nat, Nat](n.range, f).length = n
        forall(y: Nat) {
            if map[Nat, Nat](n.range, f).contains(y) {
                map_contains[Nat, Nat](n.range, f, y)
                let x: Nat satisfy {
                    n.range.contains(x) and f(x) = y
                }
                range_contains_iff_lt(n, x)
                y < n
            }
        }
        unique_nat_list_contained_by_range_length_imp_permutation(map[Nat, Nat](n.range, f), n)
        is_permutation(map[Nat, Nat](n.range, f), n.range)
    }
}

/// Pulling one summand out of a mapped sum.
theorem sum_map_remove_one[T, A: AddCommMonoid](list: List[T], item: T, f: T -> A) {
    list.contains(item) implies f(item) + sum[A](map(list.remove_one(item), f)) = sum[A](map(list, f))
} by {
    define p(l: List[T]) -> Bool {
        l.contains(item) implies f(item) + sum[A](map(l.remove_one(item), f)) = sum[A](map(l, f))
    }
    p(List.nil[T])
    forall(head: T, tail: List[T]) {
        if p(tail) {
            if List.cons(head, tail).contains(item) {
                if head = item {
                    remove_one_cons_eq(head, tail)
                    f(item) + sum[A](map(List.cons(head, tail).remove_one(item), f)) = sum[A](map(List.cons(head, tail), f))
                } else {
                    if tail.contains(item) {
                    } else {
                        false
                    }
                    f(item) + sum[A](map(tail.remove_one(item), f)) = sum[A](map(tail, f))
                    remove_one_cons_neq(head, tail, item)
                    sum[A](map(List.cons(head, tail.remove_one(item)), f)) = f(head) + sum[A](map(tail.remove_one(item), f))
                    let s = sum[A](map(tail.remove_one(item), f))
                    f(item) + (f(head) + s) = f(head) + (f(item) + s)
                    sum[A](map(List.cons(head, tail), f)) = f(head) + sum[A](map(tail, f))
                    f(item) + sum[A](map(List.cons(head, tail).remove_one(item), f)) = sum[A](map(List.cons(head, tail), f))
                }
            }
            p(List.cons(head, tail))
        }
    }
    List.induction(function(l: List[T]) { p(l) })
}

/// Removing one occurrence from a unique list yields a unique list.
theorem remove_one_unique[T](list: List[T], item: T) {
    list.is_unique implies list.remove_one(item).is_unique
} by {
    if list.is_unique {
        if not list.remove_one(item).is_unique {
            not_unique_implies_duplicate(list.remove_one(item))
            let dup: T satisfy { list.remove_one(item).count(dup) > 1 }
            list.remove_one(item).count(dup) > Nat.1
            if dup = item {
                list.remove_one(item).count(item) > Nat.1
                if list.contains(item) {
                    remove_one_count_self(list, item)
                    list.remove_one(item).count(item) + Nat.1 = list.count(item)
                    Nat.1 + Nat.1 < list.remove_one(item).count(item) + Nat.1
                    Nat.1 + Nat.1 < list.count(item)
                    Nat.1 < list.count(item)
                    unique_implies_no_duplicate(list, item)
                    false
                } else {
                    remove_one_not_contains(list, item)
                    unique_implies_no_duplicate(list, item)
                    false
                }
            } else {
                remove_one_count_other(list, item, dup)
                list.count(dup) > Nat.1
                unique_implies_no_duplicate(list, dup)
                false
            }
        }
        list.remove_one(item).is_unique
    }
}

/// In a unique list, removing one occurrence of an element drops its containment.
theorem remove_one_unique_not_contains_self[T](list: List[T], item: T) {
    list.is_unique implies not list.remove_one(item).contains(item)
} by {
    if list.is_unique {
        if list.remove_one(item).contains(item) {
            list_contains_implies_count_geq_one(list.remove_one(item), item)
            list.remove_one(item).count(item) >= Nat.1
            if list.contains(item) {
                remove_one_count_self(list, item)
                list.remove_one(item).count(item) + Nat.1 = list.count(item)
                Nat.1 + Nat.1 <= list.remove_one(item).count(item) + Nat.1
                Nat.1 + Nat.1 <= list.count(item)
                list.count(item) > Nat.1
                unique_implies_no_duplicate(list, item)
                false
            }
            remove_one_not_contains(list, item)
            list.contains(item)
            false
        }
    }
}

/// Removing an element other than the target preserves containment.
theorem remove_one_contains_other[T](list: List[T], item: T, x: T) {
    x != item implies list.contains(x) = list.remove_one(item).contains(x)
} by {
    if x != item {
        if list.contains(x) {
            list_contains_implies_count_geq_one(list, x)
            list.count(x) >= Nat.1
            remove_one_count_other(list, item, x)
            list.remove_one(item).count(x) = list.count(x)
            list.remove_one(item).count(x) >= Nat.1
            if not list.remove_one(item).contains(x) {
                list_not_contains_impl_count_zero(list.remove_one(item), x)
                list.remove_one(item).count(x) = Nat.0
                false
            }
            list.remove_one(item).contains(x)
        }
        if list.remove_one(item).contains(x) {
            list_contains_implies_count_geq_one(list.remove_one(item), x)
            list.remove_one(item).count(x) >= Nat.1
            remove_one_count_other(list, item, x)
            list.remove_one(item).count(x) = list.count(x)
            list.count(x) >= Nat.1
            if not list.contains(x) {
                list_not_contains_impl_count_zero(list, x)
                list.count(x) = Nat.0
                false
            }
            list.contains(x)
        }
        list.contains(x) = list.remove_one(item).contains(x)
    }
}

/// Two unique lists with the same membership give the same mapped sum.
theorem unique_same_contains_map_sum_eq[T, A: AddCommMonoid](a: List[T], b: List[T], f: T -> A) {
    a.is_unique and b.is_unique and (forall(x: T) { a.contains(x) = b.contains(x) })
        implies sum[A](map(a, f)) = sum[A](map(b, f))
} by {
    define p(l: List[T]) -> Bool {
        forall(other: List[T]) {
            l.is_unique and other.is_unique and (forall(x: T) { l.contains(x) = other.contains(x) })
                implies sum[A](map(l, f)) = sum[A](map(other, f))
        }
    }
    forall(other: List[T]) {
        if List.nil[T].is_unique and other.is_unique and (forall(x: T) { List.nil[T].contains(x) = other.contains(x) }) {
            forall(x: T) {
                not List.nil[T].contains(x)
                not other.contains(x)
            }
            // other has count 0 everywhere -> nil
            forall(x: T) {
                list_not_contains_impl_count_zero(other, x)
                other.count(x) = Nat.0
            }
            list_count_all_zero_imp_nil(other)
            sum[A](map(List.nil[T], f)) = sum[A](map(other, f))
        }
    }
    p(List.nil[T])
    forall(head: T, tail: List[T]) {
        if p(tail) {
            forall(other: List[T]) {
                if List.cons(head, tail).is_unique and other.is_unique and (forall(x: T) { List.cons(head, tail).contains(x) = other.contains(x) }) {
                    unique_implies_tail_unique(head, tail)
                    tail.is_unique
                    unique_implies_no_duplicate(List.cons(head, tail), head)
                    List.cons(head, tail).count(head) <= Nat.1
                    List.cons(head, tail).count(head) = Nat.1 + tail.count(head)
                    Nat.1 + tail.count(head) <= Nat.1
                    if tail.contains(head) {
                        list_contains_implies_count_geq_one(tail, head)
                        false
                    }
                    not tail.contains(head)
                    List.cons(head, tail).contains(head)
                    other.contains(head)
                    let other_rest = other.remove_one(head)
                    remove_one_unique(other, head)
                    other_rest.is_unique
                    remove_one_unique_not_contains_self(other, head)
                    not other_rest.contains(head)
                    forall(x: T) {
                        if x = head {
                            not other_rest.contains(x)
                            tail.contains(x) = other_rest.contains(x)
                        } else {
                            remove_one_contains_other(other, head, x)
                            if List.cons(head, tail).contains(x) {
                                tail.contains(x)
                            }
                            if tail.contains(x) {
                                List.cons(head, tail).contains(x)
                            }
                            List.cons(head, tail).contains(x) = tail.contains(x)
                            tail.contains(x) = other_rest.contains(x)
                        }
                    }
                    sum[A](map(tail, f)) = sum[A](map(other_rest, f))
                    sum_map_remove_one[T, A](other, head, f)
                    sum[A](map(List.cons(head, tail), f)) = f(head) + sum[A](map(tail, f))
                    sum[A](map(List.cons(head, tail), f)) = sum[A](map(other, f))
                }
            }
            p(List.cons(head, tail))
        }
    }
    List.induction(function(l: List[T]) { p(l) })
    p(a)
}

/// Mapping a permutation into an additive commutative monoid preserves its sum.
///
/// The additive twin of `permutation_preserves_mapped_product`. A sum indexed by a list is
/// therefore a function of the underlying multiset, which is what lets an index list be
/// reordered by any bijection of its members.
theorem permutation_preserves_mapped_sum[T, A: AddCommMonoid](
    a: List[T], b: List[T], f: T -> A
) {
    is_permutation(a, b) implies
        sum[A](map(a, f)) = sum[A](map(b, f))
} by {
    define p(l: List[T]) -> Bool {
        forall(other: List[T]) {
            is_permutation(l, other) implies
                sum[A](map(l, f)) = sum[A](map(other, f))
        }
    }
    forall(other: List[T]) {
        if is_permutation(List.nil[T], other) {
            permutation_symm(List.nil[T], other)
            permutation_nil_imp_nil[T](other)
            sum[A](map(List.nil[T], f)) = sum[A](map(other, f))
        }
    }
    p(List.nil[T])
    forall(head: T, tail: List[T]) {
        if p(tail) {
            forall(other: List[T]) {
                if is_permutation(List.cons(head, tail), other) {
                    List.cons(head, tail).count(head) =
                        Nat.1 + tail.count(head)
                    List.cons(head, tail).count(head) = other.count(head)
                    other.count(head) >= Nat.1
                    if not other.contains(head) {
                        list_not_contains_impl_count_zero[T](other, head)
                        other.count(head) = Nat.0
                        false
                    }
                    permutation_remove_one(
                        List.cons(head, tail), other, head)
                    remove_one_cons_eq(head, tail)
                    sum[A](map(tail, f)) =
                        sum[A](map(other.remove_one(head), f))
                    sum_map_remove_one[T, A](other, head, f)
                    f(head) + sum[A](map(other.remove_one(head), f)) =
                        sum[A](map(other, f))
                    sum[A](map(List.cons(head, tail), f)) =
                        f(head) + sum[A](map(tail, f))
                    f(head) + sum[A](map(tail, f)) =
                        f(head) + sum[A](map(other.remove_one(head), f))
                    sum[A](map(List.cons(head, tail), f)) =
                        sum[A](map(other, f))
                }
            }
            p(List.cons(head, tail))
        }
    }
    List.induction(function(l: List[T]) { p(l) })
    p(a)
}
