from list.list_base import List
from algebra.monoid.monoid import Monoid
from data.basic.functions import function_extensionality

/// Evaluates a list of generators in an arbitrary (possibly noncommutative) monoid.
define list_monoid_fold[T, A: Monoid](items: List[T], f: T -> A) -> A {
    match items {
        List.nil {
            A.1
        }
        List.cons(head, tail) {
            f(head) * list_monoid_fold(tail, f)
        }
    }
}

/// The evaluator sends the empty list to the monoid identity.
theorem list_monoid_fold_nil[T, A: Monoid](f: T -> A) {
    list_monoid_fold(List.nil[T], f) = A.1
}

/// The evaluator sends a cons list to the head value times the tail value.
theorem list_monoid_fold_cons[T, A: Monoid](head: T, tail: List[T], f: T -> A) {
    list_monoid_fold(List.cons(head, tail), f) = f(head) * list_monoid_fold(tail, f)
}

/// The evaluator sends list concatenation to monoid multiplication.
theorem list_monoid_fold_append[T, A: Monoid](left: List[T], right: List[T], f: T -> A) {
    list_monoid_fold(left + right, f) = list_monoid_fold(left, f) * list_monoid_fold(right, f)
} by {
    define p(l: List[T]) -> Bool {
        list_monoid_fold(l + right, f) = list_monoid_fold(l, f) * list_monoid_fold(right, f)
    }
    List.nil[T] + right = right
    list_monoid_fold(List.nil[T], f) = A.1
    A.1 * list_monoid_fold(right, f) = list_monoid_fold(right, f)
    p(List.nil[T])
    forall(head: T, tail: List[T]) {
        if p(tail) {
            list_monoid_fold(tail + right, f) =
                list_monoid_fold(tail, f) * list_monoid_fold(right, f)
            list_monoid_fold(List.cons(head, tail + right), f) =
                f(head) * (list_monoid_fold(tail, f) * list_monoid_fold(right, f))
            p(List.cons(head, tail))
        }
    }
    List.induction(function(l: List[T]) { p(l) })
    forall(l: List[T]) { p(l) }
}

/// The evaluator sends a singleton list to the value of its generator.
theorem list_monoid_fold_singleton[T, A: Monoid](x: T, f: T -> A) {
    list_monoid_fold(List.singleton(x), f) = f(x)
} by {
    list_monoid_fold(List.nil[T], f) = A.1
}

/// The evaluator as a function from lists to the target monoid.
define list_monoid_fold_fn[T, A: Monoid](f: T -> A) -> (List[T] -> A) {
    function(items: List[T]) {
        list_monoid_fold(items, f)
    }
}

/// A function satisfying the same nil and cons equations agrees with the evaluator at each list.
theorem list_monoid_fold_unique_at[T, A: Monoid](h: List[T] -> A, f: T -> A, xs: List[T]) {
    h(List.nil[T]) = A.1 and
    (forall(x: T, ys: List[T]) { h(List.cons(x, ys)) = f(x) * h(ys) })
    implies h(xs) = list_monoid_fold(xs, f)
} by {
    if h(List.nil[T]) = A.1 and
        forall(x: T, ys: List[T]) { h(List.cons(x, ys)) = f(x) * h(ys) } {
        define p(l: List[T]) -> Bool {
            h(l) = list_monoid_fold(l, f)
        }
        h(List.nil[T]) = A.1
        list_monoid_fold(List.nil[T], f) = A.1
        p(List.nil[T])
        forall(head: T, tail: List[T]) {
            if p(tail) {
                f(head) * h(tail) = f(head) * list_monoid_fold(tail, f)
                p(List.cons(head, tail))
            }
        }
        List.induction(function(l: List[T]) { p(l) })
        forall(l: List[T]) { p(l) }
        h(xs) = list_monoid_fold(xs, f)
    }
}

/// A function satisfying the same nil and cons equations is extensionally the evaluator.
theorem list_monoid_fold_unique[T, A: Monoid](h: List[T] -> A, f: T -> A) {
    h(List.nil[T]) = A.1 and
    (forall(x: T, ys: List[T]) { h(List.cons(x, ys)) = f(x) * h(ys) })
    implies h = list_monoid_fold_fn(f)
} by {
    if h(List.nil[T]) = A.1 and
        forall(x: T, ys: List[T]) { h(List.cons(x, ys)) = f(x) * h(ys) } {
        forall(xs: List[T]) {
            list_monoid_fold_unique_at(h, f, xs)
            h(xs) = list_monoid_fold_fn(f)(xs)
        }
        function_extensionality(h, list_monoid_fold_fn(f))
        h = list_monoid_fold_fn(f)
    }
}
