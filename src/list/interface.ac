from algebra.add_comm_monoid import AddCommMonoid
from algebra.add_semigroup import add_fn
from algebra.comm_monoid import CommMonoid
from data.basic.functions import Inhabited, compose, identity_fn, injective_fn_eq, is_injective_fn
from lattice import MeetSemilattice, JoinSemilattice, meet_lte_left, meet_lte_right,
    lte_meet_of_bounds, lte_join_left, lte_join_right, join_lte_of_bounds,
    lte_meet_iff, join_lte_iff, meet_assoc_rev, join_assoc_rev
from nat import Nat
from nat import alt_induction
from order import LinearOrder, PartialOrder, lte_refl, lte_trans, lte_min_of_bounds,
    max_lte_of_upper_bounds, min_assoc_rev, max_assoc_rev, min_lte_left,
    min_lte_right, lte_max_left, lte_max_right
from order import closed_interval
from algebra.semigroup import mul_fn
from semiring import Semiring

numerals Nat
// list_base.ac
/// A generic list data structure that can hold elements of any type.
/// Lists are constructed using nil (empty list) and cons (prepending an element).
inductive List[T] {
    /// The empty list.
    nil
    /// Constructs a list by prepending an element to an existing list.
    cons(T, List[T])
}

attributes List[T] {
    /// Concatenates two lists together.
    define add(self, other: List[T]) -> List[T] {
        match self {
            List.nil {
                other
            }
            List.cons(head, tail) {
                List.cons(head, tail.add(other))
            }
        }
    }

    /// True if this list contains the given item.
    define contains(self, item: T) -> Bool {
        match self {
            List.nil {
                false
            }
            List.cons(head, tail) {
                if head = item {
                    true
                } else {
                    tail.contains(item)
                }
            }
        }
    }
}

theorem add_nil[T](list: List[T]) {
    list + List.nil[T] = list
}

/// Concatenation of finite lists is associative.
theorem add_assoc[T](a: List[T], b: List[T], c: List[T]) {
    (a + b) + c = a + (b + c)
}

theorem add_contains_left[T](left: List[T], right: List[T], item: T) {
    left.contains(item) implies (left + right).contains(item)
}

theorem add_contains_right[T](left: List[T], right: List[T], item: T) {
    right.contains(item) implies (left + right).contains(item)
}

theorem add_contains_or[T](left: List[T], right: List[T], item: T) {
    (left + right).contains(item) implies left.contains(item) or right.contains(item)
}

theorem not_contains_add[T](left: List[T], right: List[T], item: T) {
    not left.contains(item) and not right.contains(item) implies not (left + right).contains(item)
}

attributes List[T] {
    /// True if this list contains every element of type T.
    define contains_every(self) -> Bool {
        forall(x: T) {
            self.contains(x)
        }
    }

    /// Yields the number of elements in the list.
    define length(self) -> Nat {
        match self {
            List.nil {
                Nat.0
            }
            List.cons(_, tail) {
                tail.length.suc
            }
        }
    }
}

theorem add_length[T](left: List[T], right: List[T]) {
    left.length + right.length = (left + right).length
}

attributes List[T] {
    /// Creates a list containing a single element.
    let singleton: T -> List[T] = function(x: T) {
        List.cons(x, List.nil[T])
    }

    /// Removes all duplicate elements from the list.
    /// When duplicates exist, the last occurrence is kept.
    define unique(self) -> List[T] {
        match self {
            List.nil {
                List.nil[T]
            }
            List.cons(head, tail) {
                if tail.contains(head) {
                    tail.unique
                } else {
                    List.cons(head, tail.unique)
                }
            }
        }
    }

    /// True if all elements in the list are distinct.
    define is_unique(self) -> Bool {
        self.unique = self
    }
}

theorem singleton_unique[T](item: T) {
    List.singleton(item).is_unique
}

theorem singleton_contains_imp_eq[T](item: T, x: T) {
    List.singleton(item).contains(x) implies x = item
}

theorem unique_length[T](list: List[T]) {
    list.unique.length <= list.length
}

theorem contains_imp_unique_contains[T](list: List[T], item: T) {
    list.contains(item) implies list.unique.contains(item)
}

theorem unique_contains_imp_contains[T](list: List[T], item: T) {
    list.unique.contains(item) implies list.contains(item)
}

theorem unique_preserves_contains[T](list: List[T], item: T) {
    list.unique.contains(item) = list.contains(item)
}

theorem unique_idemp[T](list: List[T]) {
    list.unique.unique = list.unique
}

theorem unique_list_is_unique[T](list: List[T]) {
    list.unique.is_unique
}

theorem unique_implies_tail_unique[T](head: T, tail: List[T]) {
    List.cons(head, tail).is_unique implies tail.is_unique
}

attributes List[T] {
    /// The number of times the given item appears in the list.
    define count(self, item: T) -> Nat {
        match self {
            List.nil[T] {
                Nat.0
            }
            List.cons(head, tail) {
                if head = item {
                    1 + tail.count(item)
                } else {
                    tail.count(item)
                }
            }
        }
    }
}

theorem list_contains_implies_count_geq_one[T](list: List[T], item: T) {
    list.contains(item) implies list.count(item) >= Nat.1
}

theorem list_not_contains_impl_count_zero[T](list: List[T], item: T) {
    not list.contains(item) implies list.count(item) = Nat.0
}

theorem unique_implies_no_duplicate[T](list: List[T], item: T) {
    list.is_unique implies list.count(item) <= Nat.1
}

theorem not_unique_implies_duplicate[T](list: List[T]) {
    not list.is_unique implies exists(x: T) {
        list.count(x) > 1
    }
}

attributes List[T] {
    /// Appends a single element to the end of the list.
    define append(self, item: T) -> List[T] {
        self + List.singleton(item)
    }
}

attributes Nat {
    /// Creates a list of natural numbers from 0 to n-1.
    define range(self) -> List[Nat] {
        match self {
            Nat.zero {
                List.nil[Nat]
            }
            Nat.suc(n) {
                n.range.append(n)
            }
        }
    }
}

attributes List[T] {
    /// Creates a list of natural numbers from 0 to n-1.
    let range: Nat -> List[Nat] = Nat.range

    /// Filters the list, keeping only elements that satisfy the given predicate.
    define filter(self, f: T -> Bool) -> List[T] {
        match self {
            List.nil {
                List.nil[T]
            }
            List.cons(head, tail) {
                if f(head) {
                    List.cons(head, tail.filter(f))
                } else {
                    tail.filter(f)
                }
            }
        }
    }

    /// Remove all instances of an element from the list.
    define remove_elem(self, elem: T) -> List[T] {
        match self {
            List.nil {
                List.nil[T]
            }
            List.cons(head, tail) {
                if head != elem {
                    List.cons(head, tail.remove_elem(elem))
                } else {
                    tail.remove_elem(elem)
                }
            }
        }
    }
}

// Filter theorems (kinda ugly, but alas)
theorem filter_only_removes_elems[T](list: List[T], f: T -> Bool) {
    list.filter(f).length <= list.length
}

theorem filter_contains_and[T](list: List[T], f: T -> Bool, item: T) {
    (list.contains(item) and f(item)) implies list.filter(f).contains(item)
}

theorem filter_contained_by_and[T](list: List[T], f: T -> Bool, item: T) {
    list.filter(f).contains(item) implies (list.contains(item) and f(item))
}

theorem filter_equivalent_to_and[T](list: List[T], f: T -> Bool, item: T) {
    (list.contains(item) and f(item)) = list.filter(f).contains(item)
}

/// Removing an element preserves exactly the other elements of the list.
theorem remove_elem_contains_iff[T](list: List[T], elem: T, item: T) {
    list.remove_elem(elem).contains(item) = (list.contains(item) and item != elem)
}

/// Removing an element leaves no occurrence of that element in the list.
theorem remove_elem_not_contains_elem[T](list: List[T], elem: T) {
    not list.remove_elem(elem).contains(elem)
}

/// Removing an element never increases the length of a list.
theorem remove_elem_length_le[T](list: List[T], elem: T) {
    list.remove_elem(elem).length <= list.length
}

/// Removing an element present in a list lowers the length of its unique reduction by one.
theorem remove_element_in_unique_equals_length_minus_one[T](list: List[T], item: T) {
    list.contains(item) implies list.unique.remove_elem(item).length + Nat.1 = list.unique.length
}

/// Removing an element is the same as filtering for elements unequal to it.
theorem remove_elem_eq_filter[T](list: List[T], elem: T) {
    list.remove_elem(elem) = list.filter(function(x: T) { x != elem })
}

/// Removing the same element twice is the same as removing it once.
theorem remove_elem_idempotent[T](list: List[T], elem: T) {
    list.remove_elem(elem).remove_elem(elem) = list.remove_elem(elem)
}

theorem filter_of_self_is_self[T](list: List[T]) {
    list.filter(list.contains) = list
}

// Length theorems
theorem length_range(n: Nat) {
    n.range.length = n
}

theorem suc_range_contains(n: Nat) {
    n.suc.range.contains(n)
}

theorem range_contains_all_leq(n: Nat) {
    forall(x: Nat) {
        x < n implies n.range.contains(x)
    }
}

theorem range_does_not_contain_geq(m: Nat, n: Nat) {
    n >= m implies not m.range.contains(n)
}

theorem lt_of_range_contains(n: Nat, x: Nat) {
    n.range.contains(x) implies x < n
}

theorem range_contains_of_lt(n: Nat, x: Nat) {
    x < n implies n.range.contains(x)
}

theorem range_contains_iff_lt(n: Nat, x: Nat) {
    n.range.contains(x) = (x < n)
}

/// Every range list `n.range` has no duplicates.
theorem range_is_unique(n: Nat) {
    n.range.is_unique
}

// And the big one: the smallest (by `.length`) list containing all elements of
// a list is its `.unique`. This now enables us to talk about "cardinality" of a
// set as being unique via `.filter(set.contains).unique` or, by the previous
// theorem, via `.unique.filter(set.contains)`.
theorem unique_is_smallest_containing_list[T](list: List[T], container: List[T]) {
    forall(x: T) {
        list.contains(x) implies container.contains(x)
    } implies list.unique.length <= container.length
}

theorem unique_list_sum[T](list1: List[T], list2: List[T]) {
    list1.is_unique and list2.is_unique and forall(x: T) {
        not (list1.contains(x) and list2.contains(x))
    } implies (list1 + list2).is_unique
}

/// Applies a function to each element of a list, creating a new list of results.
attributes List[T] {
    define map[U](self, f: T -> U) -> List[U] {
        match self {
            List.nil {
                List.nil[U]
            }
            List.cons(head, tail) {
                List.cons(f(head), tail.map(f))
            }
        }
    }
}

/// Applies a function to each element of a list, creating a new list of results.
/// Deprecated non-attribute definition.
define map[T, U](items: List[T], f: T -> U) -> List[U] {
    match items {
        List.nil {
            List.nil[U]
        }
        List.cons(head, tail) {
            List.cons(f(head), map(tail, f))
        }
    }
}

// The two definitions of map are the same
theorem map_equivalence[T, U] {
    List.map[T, U] = map[T, U]
}

/// Mapping the identity function over a list leaves the list unchanged.
theorem map_identity[T](items: List[T]) {
    map[T, T](items, identity_fn[T]) = items
}

// list_sum.ac
/// Computes the sum of all elements in a list (requires elements to form an additive commutative monoid).
define sum[A: AddCommMonoid](items: List[A]) -> A {
    match items {
        List.nil {
            A.0
        }
        List.cons(head, tail) {
            head + sum(tail)
        }
    }
}

/// Computes the partial sum of a series up to index n.
/// Returns the sum of f(0) + f(1) + ... + f(n-1).
define partial[A: AddCommMonoid](f: Nat -> A, n: Nat) -> A {
    sum(map(n.range, f))
}

theorem partial_one[A: AddCommMonoid](f: Nat -> A) {
    partial(f, 1) = f(0)
}

theorem map_sum_add[T, A: AddCommMonoid](list: List[T], f: T -> A, g: T -> A) {
    sum(map(list, f)) + sum(map(list, g)) = sum(map(list, add_fn(f, g)))
}

/// Mapped sums agree when the mapped functions agree on every list element.
theorem sum_map_of_pointwise[T, A: AddCommMonoid](items: List[T], f: T -> A, g: T -> A) {
    forall(x: T) { items.contains(x) implies f(x) = g(x) } implies
        sum(map(items, f)) = sum(map(items, g))
}

theorem partial_add[A: AddCommMonoid](f: Nat -> A, g: Nat -> A, n: Nat) {
    partial(f, n) + partial(g, n) = partial(add_fn(f, g), n)
}

/// Helper function for scalar multiplication.
define scalar_mul[S: Semiring](c: S, x: S) -> S {
    c * x
}

theorem sum_scalar_mul[S: Semiring](c: S, list: List[S]) {
    c * sum(list) = sum(map(list, scalar_mul(c)))
}

theorem sum_add[A: AddCommMonoid](left: List[A], right: List[A]) {
    sum(left + right) = sum(left) + sum(right)
}

theorem map_add[T, U](left: List[T], right: List[T], f: T -> U) {
    map(left + right, f) = map(left, f) + map(right, f)
}

theorem map_map[T, U, V](items: List[T], f: T -> U, g: U -> V) {
    map(map(items, f), g) = map(items, compose(g, f))
}

theorem map_singleton[T, U](f: T -> U, x: T) {
    map(List.singleton(x), f) = List.singleton(f(x))
}

theorem partial_scalar_mul[S: Semiring](c: S, f: Nat -> S, n: Nat) {
    c * partial(f, n) = partial(mul_fn(c, f), n)
}

theorem partial_shift_suc[A: AddCommMonoid](f: Nat -> A, n: Nat) {
    f(0) + partial(compose(f, Nat.suc), n) = partial(f, n.suc)
}

theorem partial_split_last[A: AddCommMonoid](f: Nat -> A, n: Nat) {
    partial(f, n.suc) = partial(f, n) + f(n)
}

theorem partial_pointwise_eq[A: AddCommMonoid](f: Nat -> A, g: Nat -> A, n: Nat) {
    (forall(k: Nat) { k < n implies f(k) = g(k) }) implies partial(f, n) = partial(g, n)
}

/// Helper: reverses the indices of a function over a range.
/// reverse_index(g, n, i) = g(n - i)
define reverse_index[A](g: Nat -> A, n: Nat, i: Nat) -> A {
    g(n - i)
}

theorem partial_reverse[A: AddCommMonoid](g: Nat -> A, n: Nat) {
    partial(g, n.suc) = partial(reverse_index(g, n), n.suc)
}

theorem partial_drop_first[A: AddCommMonoid](f: Nat -> A, n: Nat) {
    n > 0 implies partial(f, n) = f(0) + partial(compose(f, Nat.suc), n - 1)
}

theorem partial_split_first_last[A: AddCommMonoid](f: Nat -> A, n: Nat) {
    n >= 2 implies partial(f, n) = f(0) + partial(compose(f, Nat.suc), n - 2) + f(n - 1)
}

theorem map_length[T, U](list: List[T], f: T -> U) {
    map(list, f).length = list.length
}

theorem map_contains[T, U](list: List[T], f: T -> U, item: U) {
    map(list, f).contains(item) implies exists(x: T) {
        list.contains(x) and f(x) = item
    }
}

theorem map_contains_of_contains[T, U](list: List[T], f: T -> U, item: T) {
    list.contains(item) implies map(list, f).contains(f(item))
}

theorem pigeonhole_map_into_list[T, U](items: List[T], targets: List[U], f: T -> U) {
    items.is_unique and items.length > targets.length and
    forall(x: T) {
        items.contains(x) implies targets.contains(f(x))
    } implies exists(x: T, y: T) {
        x != y and f(x) = f(y)
    }
}

theorem injective_map_is_unique[T, U](items: List[T], f: T -> U) {
    items.is_unique and is_injective_fn(f) implies map(items, f).is_unique
}

theorem range_pigeonhole[T](n: Nat, f: Nat -> T) {
    map(n.range, f).unique.length < n implies exists (i: Nat, j: Nat) {
        i < j and j < n and f(i) = f(j)
    }
}

attributes List[T] {
    /// Yields the list without its first element.
    /// Yields nil for an empty list.
    define tail(self) -> List[T] {
        match self {
            List.nil {
                List.nil[T]
            }
            List.cons(h, t) {
                t
            }
        }
    }

    /// Removes the first n elements from the list.
    define drop(self, n: Nat) -> List[T] {
        match n {
            Nat.zero {
                self
            }
            Nat.suc(pred) {
                self.tail.drop(pred)
            }
        }
    }
}

theorem tail_cancels_cons[T](a: T, b: List[T]) {
    List.cons(a, b).tail = b
}

theorem drop_zero[T](a: List[T]) {
    a.drop(Nat.0) = a
}

theorem drop_one[T](a: List[T]) {
    a.drop(Nat.1) = a.tail
}

theorem drop_twice_all[T](m: Nat) {
    forall(l: List[T], k: Nat) {
        l.drop(m).drop(k) = l.drop(m + k)
    }
}

theorem drop_twice[T](a: List[T], m: Nat, n: Nat) {
    a.drop(m).drop(n) = a.drop(m + n)
}

attributes List[T] {
    /// Removes the last n elements from the list.
    define drop_last(self, n: Nat) -> List[T] {
        match self {
            List.nil {
                List.nil[T]
            }
            List.cons(head, tail) {
                if tail.length < n {
                    List.nil[T]
                } else {
                    List.cons(head, tail.drop_last(n))
                }
            }
        }
    }
}

attributes Nat {
    /// Creates a list of natural numbers from self to n-1 (exclusive of n).
    define until(self, n: Nat) -> List[Nat] {
        n.range.drop(self)
    }

    /// Creates a list of natural numbers from self to n (inclusive).
    define upto(self, n: Nat) -> List[Nat] {
        self.until(n.suc)
    }
}

theorem range_add_until(a: Nat, b: Nat) {
    a <= b implies a.range + a.until(b) = b.range
}

// Could be generalized to arbitrary functions over a set with 0, not
// necessarily associative as in `add`. `LinearOrder` with smallest element 0
// would similarly work (or with `option`). Technically `Nat`s w/ `max` defines a
// `Monoid`, which would also work w/ definition of `add` above.
define max_list(list: List[Nat]) -> Nat {
    match list {
        List[Nat].nil {
            0
        }
        List.cons(head, tail) {
            head.max(max_list(tail))
        }
    }
}

attributes List[T] {
    /// The index of the first occurrence of the item in the list.
    /// Returns the list length if the item is not found.
    define find_first_idx(self, item: T) -> Nat {
        match self {
            List.nil {
                Nat.0
            }
            List.cons(head, tail) {
                if head = item {
                    Nat.0
                } else {
                    1 + tail.find_first_idx(item)
                }
            }
        }
    }

    /// The element at index i, or none if the index is out of bounds.
    define get_idx(self, i: Nat) -> Option[T] {
        match self {
            List.nil {
                Option.none
            }
            List.cons(head, tail) {
                if i > 0 {
                    tail.get_idx(i - 1)
                } else {
                    Option.some(head)
                }
            }
        }
    }
}

theorem map_range[T](n: Nat, idx: Nat, f: Nat -> T) {
    idx < n implies map(n.range, f).get_idx(idx) = Option.some(f(idx))
}

theorem duplicate_implies_duplicate_idx[T](list: List[T], item: T) {
    list.count(item) > 1 implies
    exists(i: Nat, j: Nat) {
        i < j and j < list.length and
        list.get_idx(i) = Option.some(item) and
        list.get_idx(j) = Option.some(item)
    }
}

/// Helper predicate: two lists differ at some index less than n.
define differ[T](a: List[T], b: List[T], n: Nat) -> Bool {
    exists(i: Nat) {
        i < n and a.get_idx(i) != b.get_idx(i)
    }
}

theorem partial_zero[A: AddCommMonoid](f: Nat -> A) {
    partial(f, Nat.0) = A.0
}

theorem list_extensionality[T](a: List[T], b: List[T]) {
    a.length = b.length and (forall(i: Nat) { i < a.length implies a.get_idx(i) = b.get_idx(i) }) implies a = b
}

/// Theorem: map(a, f).get_idx(idx) = f(a.get_idx(idx)) if idx < a.length
theorem map_under_idx[T, U](a: List[T], f: T -> U, idx: Nat) {
    idx < a.length implies exists(x: T) {
        Option.some(x) = a.get_idx(idx) and
        map(a, f).get_idx(idx) = Option.some(f(x))
    }
}

// list_product.ac
/// Computes the product of all elements in a list (requires elements to form
/// a commutative monoid). The product of the empty list is the monoid's
/// identity element.
define product[A: CommMonoid](items: List[A]) -> A {
    match items {
        List.nil {
            A.1
        }
        List.cons(head, tail) {
            head * product(tail)
        }
    }
}

// list_filter_length.ac
theorem map_filter_length_of_pointwise[T, U](items: List[T], f: T -> U, p: U -> Bool, q: T -> Bool) {
    forall(x: T) { items.contains(x) implies p(f(x)) = q(x) } implies
    map[T, U](items, f).filter(p).length = items.filter(q).length
}

/// Filters with pointwise equal predicates have equal lengths.
theorem filter_length_of_pointwise[T](items: List[T], p: T -> Bool, q: T -> Bool) {
    forall(x: T) { items.contains(x) implies p(x) = q(x) } implies
        items.filter(p).length = items.filter(q).length
}

/// Filtering by a false predicate produces an empty-length list.
theorem filter_false_length[T](items: List[T]) {
    items.filter(function(x: T) { false }).length = Nat.0
}

/// Consing a fresh element onto a unique list gives a unique list.
theorem cons_unique_of_tail_unique_not_contains[T](head: T, tail: List[T]) {
    tail.is_unique and not tail.contains(head) implies List.cons(head, tail).is_unique
}

/// Filtering distributes over list concatenation.
theorem filter_add[T](left: List[T], right: List[T], pred: T -> Bool) {
    (left + right).filter(pred) = left.filter(pred) + right.filter(pred)
}

/// The length of a filtered concatenation is the sum of filtered lengths.
theorem filter_add_length[T](left: List[T], right: List[T], pred: T -> Bool) {
    (left + right).filter(pred).length = left.filter(pred).length + right.filter(pred).length
}

/// Lists with the same elements have filters with the same membership.
theorem same_contains_filter_contains_iff[T](a: List[T], b: List[T], pred: T -> Bool, x: T) {
    (forall(y: T) { a.contains(y) = b.contains(y) }) implies
    a.filter(pred).contains(x) = b.filter(pred).contains(x)
}

/// Filtering a unique list preserves uniqueness.
theorem filter_preserves_unique[T](list: List[T], pred: T -> Bool) {
    list.is_unique implies list.filter(pred).is_unique
}

/// Unique lists with the same elements have equal filtered lengths.
theorem unique_same_contains_filter_length[T](a: List[T], b: List[T], pred: T -> Bool) {
    a.is_unique and b.is_unique and
    (forall(x: T) { a.contains(x) = b.contains(x) })
        implies a.filter(pred).length = b.filter(pred).length
}

// list_lattice.ac
/// The meet of a non-empty list.
define list_meet[S: MeetSemilattice](head: S, tail: List[S]) -> S {
    match tail {
        List.nil {
            head
        }
        List.cons(next, rest) {
            head.meet(list_meet(next, rest))
        }
    }
}

/// The join of a non-empty list.
define list_join[S: JoinSemilattice](head: S, tail: List[S]) -> S {
    match tail {
        List.nil {
            head
        }
        List.cons(next, rest) {
            head.join(list_join(next, rest))
        }
    }
}

/// The infimum of a non-empty list.
define list_inf[S: MeetSemilattice](head: S, tail: List[S]) -> S {
    list_meet(head, tail)
}

/// The supremum of a non-empty list.
define list_sup[S: JoinSemilattice](head: S, tail: List[S]) -> S {
    list_join(head, tail)
}

/// True if a point is below every element of a list.
define list_lower_bound[S: PartialOrder](items: List[S], lower: S) -> Bool {
    forall(x: S) {
        items.contains(x) implies lower <= x
    }
}

/// True if a point is above every element of a list.
define list_upper_bound[S: PartialOrder](items: List[S], upper: S) -> Bool {
    forall(x: S) {
        items.contains(x) implies x <= upper
    }
}

theorem list_lower_bound_cons_iff[S: PartialOrder](head: S, tail: List[S], lower: S) {
    list_lower_bound(List.cons(head, tail), lower) =
    (lower <= head and list_lower_bound(tail, lower))
}

theorem list_upper_bound_cons_iff[S: PartialOrder](head: S, tail: List[S], upper: S) {
    list_upper_bound(List.cons(head, tail), upper) =
    (head <= upper and list_upper_bound(tail, upper))
}

theorem list_lower_bound_add_left[S: PartialOrder](left: List[S], right: List[S], lower: S) {
    list_lower_bound(left + right, lower) implies list_lower_bound(left, lower)
}

theorem list_lower_bound_add_right[S: PartialOrder](left: List[S], right: List[S], lower: S) {
    list_lower_bound(left + right, lower) implies list_lower_bound(right, lower)
}

theorem list_lower_bound_add_iff[S: PartialOrder](left: List[S], right: List[S], lower: S) {
    list_lower_bound(left + right, lower) =
    (list_lower_bound(left, lower) and list_lower_bound(right, lower))
}

theorem list_upper_bound_add_left[S: PartialOrder](left: List[S], right: List[S], upper: S) {
    list_upper_bound(left + right, upper) implies list_upper_bound(left, upper)
}

theorem list_upper_bound_add_right[S: PartialOrder](left: List[S], right: List[S], upper: S) {
    list_upper_bound(left + right, upper) implies list_upper_bound(right, upper)
}

theorem list_upper_bound_add_iff[S: PartialOrder](left: List[S], right: List[S], upper: S) {
    list_upper_bound(left + right, upper) =
    (list_upper_bound(left, upper) and list_upper_bound(right, upper))
}

theorem list_lower_bound_nil[S: PartialOrder](lower: S) {
    list_lower_bound(List.nil[S], lower)
}

theorem list_upper_bound_nil[S: PartialOrder](upper: S) {
    list_upper_bound(List.nil[S], upper)
}

theorem list_lower_bound_singleton_iff[S: PartialOrder](item: S, lower: S) {
    list_lower_bound(List.singleton(item), lower) = (lower <= item)
}

theorem list_upper_bound_singleton_iff[S: PartialOrder](item: S, upper: S) {
    list_upper_bound(List.singleton(item), upper) = (item <= upper)
}

theorem list_lower_bound_append_iff[S: PartialOrder](items: List[S], last: S, lower: S) {
    list_lower_bound(items.append(last), lower) =
    (list_lower_bound(items, lower) and lower <= last)
}

theorem list_upper_bound_append_iff[S: PartialOrder](items: List[S], last: S, upper: S) {
    list_upper_bound(items.append(last), upper) =
    (list_upper_bound(items, upper) and last <= upper)
}

theorem list_lower_bound_monotone[S: PartialOrder](items: List[S], lower: S, smaller: S) {
    list_lower_bound(items, lower) and smaller <= lower implies list_lower_bound(items, smaller)
}

theorem list_upper_bound_monotone[S: PartialOrder](items: List[S], upper: S, larger: S) {
    list_upper_bound(items, upper) and upper <= larger implies list_upper_bound(items, larger)
}

theorem list_meet_lte_tail_meet[S: MeetSemilattice](head: S, next: S, rest: List[S]) {
    list_meet(head, List.cons(next, rest)) <= list_meet(next, rest)
}

theorem tail_join_lte_list_join[S: JoinSemilattice](head: S, next: S, rest: List[S]) {
    list_join(next, rest) <= list_join(head, List.cons(next, rest))
}

theorem list_meet_is_lower_bound[S: MeetSemilattice](head: S, tail: List[S]) {
    list_lower_bound(List.cons(head, tail), list_meet(head, tail))
}

theorem le_list_meet_iff[S: MeetSemilattice](c: S, head: S, tail: List[S]) {
    c <= list_meet(head, tail) = list_lower_bound(List.cons(head, tail), c)
}

theorem list_join_is_upper_bound[S: JoinSemilattice](head: S, tail: List[S]) {
    list_upper_bound(List.cons(head, tail), list_join(head, tail))
}

theorem list_join_le_iff[S: JoinSemilattice](head: S, tail: List[S], c: S) {
    list_join(head, tail) <= c = list_upper_bound(List.cons(head, tail), c)
}

theorem list_meet_add_cons[S: MeetSemilattice](head: S, tail: List[S], next: S, rest: List[S]) {
    list_meet(head, tail + List.cons(next, rest)) =
    list_meet(head, tail).meet(list_meet(next, rest))
}

theorem list_join_add_cons[S: JoinSemilattice](head: S, tail: List[S], next: S, rest: List[S]) {
    list_join(head, tail + List.cons(next, rest)) =
    list_join(head, tail).join(list_join(next, rest))
}

theorem list_meet_add_cons_lte_left[S: MeetSemilattice](head: S, tail: List[S], next: S, rest: List[S]) {
    list_meet(head, tail + List.cons(next, rest)) <= list_meet(head, tail)
}

theorem list_meet_add_cons_lte_right[S: MeetSemilattice](head: S, tail: List[S], next: S, rest: List[S]) {
    list_meet(head, tail + List.cons(next, rest)) <= list_meet(next, rest)
}

theorem list_join_lte_add_cons_left[S: JoinSemilattice](head: S, tail: List[S], next: S, rest: List[S]) {
    list_join(head, tail) <= list_join(head, tail + List.cons(next, rest))
}

theorem list_join_lte_add_cons_right[S: JoinSemilattice](head: S, tail: List[S], next: S, rest: List[S]) {
    list_join(next, rest) <= list_join(head, tail + List.cons(next, rest))
}

theorem le_list_meet_add_cons_iff[S: MeetSemilattice](c: S, head: S, tail: List[S],
        next: S, rest: List[S]) {
    c <= list_meet(head, tail + List.cons(next, rest)) =
    (c <= list_meet(head, tail) and c <= list_meet(next, rest))
}

theorem list_join_add_cons_le_iff[S: JoinSemilattice](head: S, tail: List[S],
        next: S, rest: List[S], c: S) {
    list_join(head, tail + List.cons(next, rest)) <= c =
    (list_join(head, tail) <= c and list_join(next, rest) <= c)
}

theorem list_meet_append[S: MeetSemilattice](head: S, tail: List[S], last: S) {
    list_meet(head, tail.append(last)) = list_meet(head, tail).meet(last)
}

theorem list_join_append[S: JoinSemilattice](head: S, tail: List[S], last: S) {
    list_join(head, tail.append(last)) = list_join(head, tail).join(last)
}

theorem list_meet_append_lte_list_meet[S: MeetSemilattice](head: S, tail: List[S], last: S) {
    list_meet(head, tail.append(last)) <= list_meet(head, tail)
}

theorem list_meet_append_lte_last[S: MeetSemilattice](head: S, tail: List[S], last: S) {
    list_meet(head, tail.append(last)) <= last
}

theorem list_join_lte_append_join[S: JoinSemilattice](head: S, tail: List[S], last: S) {
    list_join(head, tail) <= list_join(head, tail.append(last))
}

theorem last_lte_list_join_append[S: JoinSemilattice](head: S, tail: List[S], last: S) {
    last <= list_join(head, tail.append(last))
}

theorem le_list_meet_append_iff[S: MeetSemilattice](c: S, head: S, tail: List[S], last: S) {
    c <= list_meet(head, tail.append(last)) =
    (c <= list_meet(head, tail) and c <= last)
}

theorem list_join_append_le_iff[S: JoinSemilattice](head: S, tail: List[S], last: S, c: S) {
    list_join(head, tail.append(last)) <= c =
    (list_join(head, tail) <= c and last <= c)
}

theorem list_inf_eq_list_meet[S: MeetSemilattice](head: S, tail: List[S]) {
    list_inf(head, tail) = list_meet(head, tail)
}

theorem list_sup_eq_list_join[S: JoinSemilattice](head: S, tail: List[S]) {
    list_sup(head, tail) = list_join(head, tail)
}

theorem list_inf_lte_head[S: MeetSemilattice](head: S, tail: List[S]) {
    list_inf(head, tail) <= head
}

theorem list_inf_lte_tail_inf[S: MeetSemilattice](head: S, next: S, rest: List[S]) {
    list_inf(head, List.cons(next, rest)) <= list_inf(next, rest)
}

theorem list_inf_lte_contains[S: MeetSemilattice](head: S, tail: List[S], x: S) {
    List.cons(head, tail).contains(x) implies list_inf(head, tail) <= x
}

theorem head_lte_list_sup[S: JoinSemilattice](head: S, tail: List[S]) {
    head <= list_sup(head, tail)
}

theorem tail_sup_lte_list_sup[S: JoinSemilattice](head: S, next: S, rest: List[S]) {
    list_sup(next, rest) <= list_sup(head, List.cons(next, rest))
}

theorem contains_lte_list_sup[S: JoinSemilattice](head: S, tail: List[S], x: S) {
    List.cons(head, tail).contains(x) implies x <= list_sup(head, tail)
}

theorem lte_list_inf_of_contains_bounds[S: MeetSemilattice](c: S, head: S, tail: List[S]) {
    list_lower_bound(List.cons(head, tail), c) implies
    c <= list_inf(head, tail)
}

theorem lte_list_inf_iff_contains_bounds[S: MeetSemilattice](c: S, head: S, tail: List[S]) {
    c <= list_inf(head, tail) = list_lower_bound(List.cons(head, tail), c)
}

theorem list_inf_is_lower_bound[S: MeetSemilattice](head: S, tail: List[S]) {
    list_lower_bound(List.cons(head, tail), list_inf(head, tail))
}

theorem le_list_inf_iff[S: MeetSemilattice](c: S, head: S, tail: List[S]) {
    c <= list_inf(head, tail) = list_lower_bound(List.cons(head, tail), c)
}

theorem list_sup_lte_of_contains_bounds[S: JoinSemilattice](head: S, tail: List[S], c: S) {
    list_upper_bound(List.cons(head, tail), c) implies
    list_sup(head, tail) <= c
}

theorem list_sup_lte_iff_contains_bounds[S: JoinSemilattice](head: S, tail: List[S], c: S) {
    list_sup(head, tail) <= c = list_upper_bound(List.cons(head, tail), c)
}

theorem list_sup_is_upper_bound[S: JoinSemilattice](head: S, tail: List[S]) {
    list_upper_bound(List.cons(head, tail), list_sup(head, tail))
}

theorem list_sup_le_iff[S: JoinSemilattice](head: S, tail: List[S], c: S) {
    list_sup(head, tail) <= c = list_upper_bound(List.cons(head, tail), c)
}

theorem list_inf_add_cons[S: MeetSemilattice](head: S, tail: List[S], next: S, rest: List[S]) {
    list_inf(head, tail + List.cons(next, rest)) =
    list_inf(head, tail).meet(list_inf(next, rest))
}

theorem list_sup_add_cons[S: JoinSemilattice](head: S, tail: List[S], next: S, rest: List[S]) {
    list_sup(head, tail + List.cons(next, rest)) =
    list_sup(head, tail).join(list_sup(next, rest))
}

theorem list_inf_add_cons_lte_left[S: MeetSemilattice](head: S, tail: List[S], next: S, rest: List[S]) {
    list_inf(head, tail + List.cons(next, rest)) <= list_inf(head, tail)
}

theorem list_inf_add_cons_lte_right[S: MeetSemilattice](head: S, tail: List[S], next: S, rest: List[S]) {
    list_inf(head, tail + List.cons(next, rest)) <= list_inf(next, rest)
}

theorem list_sup_lte_add_cons_left[S: JoinSemilattice](head: S, tail: List[S], next: S, rest: List[S]) {
    list_sup(head, tail) <= list_sup(head, tail + List.cons(next, rest))
}

theorem list_sup_lte_add_cons_right[S: JoinSemilattice](head: S, tail: List[S], next: S, rest: List[S]) {
    list_sup(next, rest) <= list_sup(head, tail + List.cons(next, rest))
}

theorem le_list_inf_add_cons_iff[S: MeetSemilattice](c: S, head: S, tail: List[S],
        next: S, rest: List[S]) {
    c <= list_inf(head, tail + List.cons(next, rest)) =
    (c <= list_inf(head, tail) and c <= list_inf(next, rest))
}

theorem list_sup_add_cons_le_iff[S: JoinSemilattice](head: S, tail: List[S],
        next: S, rest: List[S], c: S) {
    list_sup(head, tail + List.cons(next, rest)) <= c =
    (list_sup(head, tail) <= c and list_sup(next, rest) <= c)
}

theorem list_inf_append[S: MeetSemilattice](head: S, tail: List[S], last: S) {
    list_inf(head, tail.append(last)) = list_inf(head, tail).meet(last)
}

theorem list_sup_append[S: JoinSemilattice](head: S, tail: List[S], last: S) {
    list_sup(head, tail.append(last)) = list_sup(head, tail).join(last)
}

theorem list_inf_append_lte_list_inf[S: MeetSemilattice](head: S, tail: List[S], last: S) {
    list_inf(head, tail.append(last)) <= list_inf(head, tail)
}

theorem list_inf_append_lte_last[S: MeetSemilattice](head: S, tail: List[S], last: S) {
    list_inf(head, tail.append(last)) <= last
}

theorem list_sup_lte_append_sup[S: JoinSemilattice](head: S, tail: List[S], last: S) {
    list_sup(head, tail) <= list_sup(head, tail.append(last))
}

theorem last_lte_list_sup_append[S: JoinSemilattice](head: S, tail: List[S], last: S) {
    last <= list_sup(head, tail.append(last))
}

theorem le_list_inf_append_iff[S: MeetSemilattice](c: S, head: S, tail: List[S], last: S) {
    c <= list_inf(head, tail.append(last)) =
    (c <= list_inf(head, tail) and c <= last)
}

theorem list_sup_append_le_iff[S: JoinSemilattice](head: S, tail: List[S], last: S, c: S) {
    list_sup(head, tail.append(last)) <= c =
    (list_sup(head, tail) <= c and last <= c)
}

theorem list_meet_singleton[S: MeetSemilattice](head: S) {
    list_meet(head, List.nil[S]) = head
}

theorem list_join_singleton[S: JoinSemilattice](head: S) {
    list_join(head, List.nil[S]) = head
}

theorem list_inf_singleton[S: MeetSemilattice](head: S) {
    list_inf(head, List.nil[S]) = head
}

theorem list_sup_singleton[S: JoinSemilattice](head: S) {
    list_sup(head, List.nil[S]) = head
}

theorem list_meet_cons[S: MeetSemilattice](head: S, next: S, rest: List[S]) {
    list_meet(head, List.cons(next, rest)) = head.meet(list_meet(next, rest))
}

theorem list_join_cons[S: JoinSemilattice](head: S, next: S, rest: List[S]) {
    list_join(head, List.cons(next, rest)) = head.join(list_join(next, rest))
}

theorem list_inf_cons[S: MeetSemilattice](head: S, next: S, rest: List[S]) {
    list_inf(head, List.cons(next, rest)) = head.meet(list_inf(next, rest))
}

theorem list_sup_cons[S: JoinSemilattice](head: S, next: S, rest: List[S]) {
    list_sup(head, List.cons(next, rest)) = head.join(list_sup(next, rest))
}

theorem list_meet_pair[S: MeetSemilattice](a: S, b: S) {
    list_meet(a, List.cons(b, List.nil[S])) = a.meet(b)
}

theorem list_join_pair[S: JoinSemilattice](a: S, b: S) {
    list_join(a, List.cons(b, List.nil[S])) = a.join(b)
}

theorem list_inf_pair[S: MeetSemilattice](a: S, b: S) {
    list_inf(a, List.cons(b, List.nil[S])) = a.meet(b)
}

theorem list_sup_pair[S: JoinSemilattice](a: S, b: S) {
    list_sup(a, List.cons(b, List.nil[S])) = a.join(b)
}

theorem le_list_inf_cons_of_le_head_of_le_tail[S: MeetSemilattice](c: S, head: S, next: S, rest: List[S]) {
    c <= head and c <= list_inf(next, rest) implies c <= list_inf(head, List.cons(next, rest))
}

theorem list_sup_cons_le_of_head_le_of_tail_le[S: JoinSemilattice](head: S, next: S, rest: List[S], c: S) {
    head <= c and list_sup(next, rest) <= c implies list_sup(head, List.cons(next, rest)) <= c
}

theorem le_list_inf_cons_iff[S: MeetSemilattice](c: S, head: S, next: S, rest: List[S]) {
    c <= list_inf(head, List.cons(next, rest)) =
    (c <= head and c <= list_inf(next, rest))
}

theorem list_sup_cons_le_iff[S: JoinSemilattice](head: S, next: S, rest: List[S], c: S) {
    list_sup(head, List.cons(next, rest)) <= c =
    (head <= c and list_sup(next, rest) <= c)
}

theorem le_list_inf_pair_iff[S: MeetSemilattice](c: S, a: S, b: S) {
    c <= list_inf(a, List.cons(b, List.nil[S])) = (c <= a and c <= b)
}

theorem list_sup_pair_le_iff[S: JoinSemilattice](a: S, b: S, c: S) {
    list_sup(a, List.cons(b, List.nil[S])) <= c = (a <= c and b <= c)
}

// list_order.ac
/// Theorems about lists of ordered types.

/// True if t is an upper bound for all items in the list.
/// An upper bound t satisfies: every element in the list is ≤ t.
define is_upper_bound[T: LinearOrder](list: List[T], t: T) -> Bool {
    match list {
        List.nil {
            true
        }
        List.cons(head, tail) {
            head <= t and is_upper_bound(tail, t)
        }
    }
}

theorem upper_bound_contains[T: LinearOrder](list: List[T], ub: T, c: T) {
    is_upper_bound(list, ub) and list.contains(c) implies c <= ub
}

theorem upper_bound_add_left[T: LinearOrder](list1: List[T], list2: List[T], ub: T) {
    is_upper_bound(list1 + list2, ub) implies is_upper_bound(list1, ub)
}

theorem upper_bound_add_right[T: LinearOrder](list1: List[T], list2: List[T], ub: T) {
    is_upper_bound(list1 + list2, ub) implies is_upper_bound(list2, ub)
}

theorem upper_bound_monotone[T: LinearOrder](list: List[T], ub1: T, ub2: T) {
    is_upper_bound(list, ub1) and ub1 <= ub2 implies is_upper_bound(list, ub2)
}

theorem upper_bound_iff_list_upper_bound[T: LinearOrder](list: List[T], upper: T) {
    is_upper_bound(list, upper) = list_upper_bound(list, upper)
}

theorem upper_bound_imp_list_upper_bound[T: LinearOrder](list: List[T], upper: T) {
    is_upper_bound(list, upper) implies list_upper_bound(list, upper)
}

theorem list_upper_bound_imp_upper_bound[T: LinearOrder](list: List[T], upper: T) {
    list_upper_bound(list, upper) implies is_upper_bound(list, upper)
}

theorem upper_bound_nil[T: LinearOrder](upper: T) {
    is_upper_bound(List.nil[T], upper)
}

theorem upper_bound_cons_iff[T: LinearOrder](head: T, tail: List[T], upper: T) {
    is_upper_bound(List.cons(head, tail), upper) =
    (head <= upper and is_upper_bound(tail, upper))
}

theorem upper_bound_add_iff[T: LinearOrder](left: List[T], right: List[T], upper: T) {
    is_upper_bound(left + right, upper) =
    (is_upper_bound(left, upper) and is_upper_bound(right, upper))
}

theorem upper_bound_singleton_iff[T: LinearOrder](item: T, upper: T) {
    is_upper_bound(List.singleton(item), upper) = (item <= upper)
}

theorem upper_bound_append_iff[T: LinearOrder](items: List[T], last: T, upper: T) {
    is_upper_bound(items.append(last), upper) =
    (is_upper_bound(items, upper) and last <= upper)
}

/// The maximum element of a non-empty list.
define list_max[T: LinearOrder](head: T, tail: List[T]) -> T {
    match tail {
        List.nil {
            head
        }
        List.cons(next, rest) {
            head.max(list_max(next, rest))
        }
    }
}

theorem list_max_is_upper_bound[T: LinearOrder](head: T, tail: List[T]) {
    is_upper_bound(List.cons(head, tail), list_max(head, tail))
}

theorem list_max_lte_of_upper_bound[T: LinearOrder](head: T, tail: List[T], upper: T) {
    is_upper_bound(List.cons(head, tail), upper) implies
    list_max(head, tail) <= upper
}

theorem list_max_lte_iff_upper_bound[T: LinearOrder](head: T, tail: List[T], upper: T) {
    list_max(head, tail) <= upper = is_upper_bound(List.cons(head, tail), upper)
}

theorem list_max_le_iff[T: LinearOrder](head: T, tail: List[T], upper: T) {
    list_max(head, tail) <= upper = is_upper_bound(List.cons(head, tail), upper)
}

theorem list_max_is_list_upper_bound[T: LinearOrder](head: T, tail: List[T]) {
    list_upper_bound(List.cons(head, tail), list_max(head, tail))
}

theorem list_max_le_iff_list_upper_bound[T: LinearOrder](head: T, tail: List[T], upper: T) {
    list_max(head, tail) <= upper = list_upper_bound(List.cons(head, tail), upper)
}

/// The maximum of a non-empty list belongs to the list.
theorem list_max_contains[T: LinearOrder](head: T, tail: List[T]) {
    List.cons(head, tail).contains(list_max(head, tail))
}

theorem contains_lte_list_max[T: LinearOrder](head: T, tail: List[T], item: T) {
    List.cons(head, tail).contains(item) implies item <= list_max(head, tail)
}

/// A list containing an element has a greatest element.
theorem list_contains_has_greatest[T: LinearOrder](items: List[T], witness: T) {
    items.contains(witness) implies exists(greatest: T) {
        items.contains(greatest) and forall(item: T) {
            items.contains(item) implies item <= greatest
        }
    }
}

/// A nonempty list in a partial order has a maximal element.
theorem list_cons_has_maximal[T: PartialOrder](candidate: T, items: List[T]) {
    exists(maximal: T) {
        List.cons(candidate, items).contains(maximal) and forall(item: T) {
            List.cons(candidate, items).contains(item) and maximal <= item implies item = maximal
        }
    }
}

/// A list containing an element has a maximal element in a partial order.
theorem list_contains_has_maximal[T: PartialOrder](items: List[T], witness: T) {
    items.contains(witness) implies exists(maximal: T) {
        items.contains(maximal) and forall(item: T) {
            items.contains(item) and maximal <= item implies item = maximal
        }
    }
}

/// A nonempty list in a partial order has a minimal element.
theorem list_cons_has_minimal[T: PartialOrder](candidate: T, items: List[T]) {
    exists(minimal: T) {
        List.cons(candidate, items).contains(minimal) and forall(item: T) {
            List.cons(candidate, items).contains(item) and item <= minimal implies item = minimal
        }
    }
}

/// A list containing an element has a minimal element in a partial order.
theorem list_contains_has_minimal[T: PartialOrder](items: List[T], witness: T) {
    items.contains(witness) implies exists(minimal: T) {
        items.contains(minimal) and forall(item: T) {
            items.contains(item) and item <= minimal implies item = minimal
        }
    }
}

theorem head_lte_list_max[T: LinearOrder](head: T, tail: List[T]) {
    head <= list_max(head, tail)
}

theorem list_has_upper_bound[T: LinearOrder](head: T, tail: List[T]) {
    exists(ub: T) {
        is_upper_bound(List.cons(head, tail), ub)
    }
}

/// True if t is a lower bound for all items in the list.
/// A lower bound t satisfies: t ≤ every element in the list.
define is_lower_bound[T: LinearOrder](list: List[T], t: T) -> Bool {
    match list {
        List.nil {
            true
        }
        List.cons(head, tail) {
            t <= head and is_lower_bound(tail, t)
        }
    }
}

theorem lower_bound_contains[T: LinearOrder](list: List[T], lb: T, c: T) {
    is_lower_bound(list, lb) and list.contains(c) implies lb <= c
}

theorem lower_bound_add_left[T: LinearOrder](list1: List[T], list2: List[T], lb: T) {
    is_lower_bound(list1 + list2, lb) implies is_lower_bound(list1, lb)
}

theorem lower_bound_add_right[T: LinearOrder](list1: List[T], list2: List[T], lb: T) {
    is_lower_bound(list1 + list2, lb) implies is_lower_bound(list2, lb)
}

theorem lower_bound_monotone[T: LinearOrder](list: List[T], lb1: T, lb2: T) {
    is_lower_bound(list, lb1) and lb2 <= lb1 implies is_lower_bound(list, lb2)
}

theorem lower_bound_iff_list_lower_bound[T: LinearOrder](list: List[T], lower: T) {
    is_lower_bound(list, lower) = list_lower_bound(list, lower)
}

theorem lower_bound_imp_list_lower_bound[T: LinearOrder](list: List[T], lower: T) {
    is_lower_bound(list, lower) implies list_lower_bound(list, lower)
}

theorem list_lower_bound_imp_lower_bound[T: LinearOrder](list: List[T], lower: T) {
    list_lower_bound(list, lower) implies is_lower_bound(list, lower)
}

theorem lower_bound_nil[T: LinearOrder](lower: T) {
    is_lower_bound(List.nil[T], lower)
}

theorem lower_bound_cons_iff[T: LinearOrder](head: T, tail: List[T], lower: T) {
    is_lower_bound(List.cons(head, tail), lower) =
    (lower <= head and is_lower_bound(tail, lower))
}

theorem lower_bound_add_iff[T: LinearOrder](left: List[T], right: List[T], lower: T) {
    is_lower_bound(left + right, lower) =
    (is_lower_bound(left, lower) and is_lower_bound(right, lower))
}

theorem lower_bound_singleton_iff[T: LinearOrder](item: T, lower: T) {
    is_lower_bound(List.singleton(item), lower) = (lower <= item)
}

theorem lower_bound_append_iff[T: LinearOrder](items: List[T], last: T, lower: T) {
    is_lower_bound(items.append(last), lower) =
    (is_lower_bound(items, lower) and lower <= last)
}

/// True if all elements of a list lie in the closed interval from `lower` to `upper`.
define is_interval_bound[T: LinearOrder](list: List[T], lower: T, upper: T) -> Bool {
    is_lower_bound(list, lower) and is_upper_bound(list, upper)
}

theorem interval_bound_contains[T: LinearOrder](list: List[T], lower: T, upper: T, item: T) {
    is_interval_bound(list, lower, upper) and list.contains(item) implies
    closed_interval(lower, upper, item)
}

theorem bounds_imp_interval_bound[T: LinearOrder](list: List[T], lower: T, upper: T) {
    is_lower_bound(list, lower) and is_upper_bound(list, upper) implies
    is_interval_bound(list, lower, upper)
}

theorem interval_bound_imp_lower_bound[T: LinearOrder](list: List[T], lower: T, upper: T) {
    is_interval_bound(list, lower, upper) implies is_lower_bound(list, lower)
}

theorem interval_bound_imp_upper_bound[T: LinearOrder](list: List[T], lower: T, upper: T) {
    is_interval_bound(list, lower, upper) implies is_upper_bound(list, upper)
}

theorem interval_bound_iff_bounds[T: LinearOrder](list: List[T], lower: T, upper: T) {
    is_interval_bound(list, lower, upper) =
    (is_lower_bound(list, lower) and is_upper_bound(list, upper))
}

theorem interval_bound_nil[T: LinearOrder](lower: T, upper: T) {
    is_interval_bound(List.nil[T], lower, upper)
}

theorem interval_bound_cons_iff[T: LinearOrder](head: T, tail: List[T], lower: T, upper: T) {
    is_interval_bound(List.cons(head, tail), lower, upper) =
    (closed_interval(lower, upper, head) and is_interval_bound(tail, lower, upper))
}

theorem interval_bound_add_iff[T: LinearOrder](left: List[T], right: List[T],
        lower: T, upper: T) {
    is_interval_bound(left + right, lower, upper) =
    (is_interval_bound(left, lower, upper) and is_interval_bound(right, lower, upper))
}

theorem interval_bound_singleton_iff[T: LinearOrder](item: T, lower: T, upper: T) {
    is_interval_bound(List.singleton(item), lower, upper) =
    closed_interval(lower, upper, item)
}

theorem interval_bound_append_iff[T: LinearOrder](items: List[T], last: T,
        lower: T, upper: T) {
    is_interval_bound(items.append(last), lower, upper) =
    (is_interval_bound(items, lower, upper) and closed_interval(lower, upper, last))
}

theorem lower_bound_imp_contains_lower_bound[T: LinearOrder](list: List[T], lower: T) {
    is_lower_bound(list, lower) implies
    lib(order).is_lower_bound(list.contains, lower)
}

theorem contains_lower_bound_imp_lower_bound[T: LinearOrder](list: List[T], lower: T) {
    lib(order).is_lower_bound(list.contains, lower) implies
    is_lower_bound(list, lower)
}

theorem lower_bound_iff_contains_lower_bound[T: LinearOrder](list: List[T], lower: T) {
    is_lower_bound(list, lower) =
    lib(order).is_lower_bound(list.contains, lower)
}

theorem lower_bound_imp_contains_bounded_below[T: LinearOrder](list: List[T], lower: T) {
    is_lower_bound(list, lower) implies
    lib(order).is_bounded_below(list.contains)
}

theorem upper_bound_imp_contains_upper_bound[T: LinearOrder](list: List[T], upper: T) {
    is_upper_bound(list, upper) implies
    lib(order).is_upper_bound(list.contains, upper)
}

theorem contains_upper_bound_imp_upper_bound[T: LinearOrder](list: List[T], upper: T) {
    lib(order).is_upper_bound(list.contains, upper) implies
    is_upper_bound(list, upper)
}

theorem upper_bound_iff_contains_upper_bound[T: LinearOrder](list: List[T], upper: T) {
    is_upper_bound(list, upper) =
    lib(order).is_upper_bound(list.contains, upper)
}

theorem upper_bound_imp_contains_bounded_above[T: LinearOrder](list: List[T], upper: T) {
    is_upper_bound(list, upper) implies
    lib(order).is_bounded_above(list.contains)
}

theorem interval_bound_imp_contains_bounded_by_interval[T: LinearOrder](list: List[T],
        lower: T, upper: T) {
    is_interval_bound(list, lower, upper) implies
    lib(order).is_bounded_by_interval(list.contains, lower, upper)
}

theorem contains_bounded_by_interval_imp_interval_bound[T: LinearOrder](list: List[T],
        lower: T, upper: T) {
    lib(order).is_bounded_by_interval(list.contains, lower, upper) implies
    is_interval_bound(list, lower, upper)
}

theorem interval_bound_iff_contains_bounded_by_interval[T: LinearOrder](list: List[T],
        lower: T, upper: T) {
    is_interval_bound(list, lower, upper) =
    lib(order).is_bounded_by_interval(list.contains, lower, upper)
}

theorem interval_bound_imp_contains_bounded[T: LinearOrder](list: List[T],
        lower: T, upper: T) {
    is_interval_bound(list, lower, upper) implies
    lib(order).is_bounded(list.contains)
}

/// The minimum element of a non-empty list.
define list_min[T: LinearOrder](head: T, tail: List[T]) -> T {
    match tail {
        List.nil {
            head
        }
        List.cons(next, rest) {
            head.min(list_min(next, rest))
        }
    }
}

theorem list_min_is_lower_bound[T: LinearOrder](head: T, tail: List[T]) {
    is_lower_bound(List.cons(head, tail), list_min(head, tail))
}

theorem lte_list_min_of_lower_bound[T: LinearOrder](lower: T, head: T, tail: List[T]) {
    is_lower_bound(List.cons(head, tail), lower) implies
    lower <= list_min(head, tail)
}

theorem lte_list_min_iff_lower_bound[T: LinearOrder](lower: T, head: T, tail: List[T]) {
    lower <= list_min(head, tail) = is_lower_bound(List.cons(head, tail), lower)
}

theorem le_list_min_iff[T: LinearOrder](lower: T, head: T, tail: List[T]) {
    lower <= list_min(head, tail) = is_lower_bound(List.cons(head, tail), lower)
}

theorem list_min_is_list_lower_bound[T: LinearOrder](head: T, tail: List[T]) {
    list_lower_bound(List.cons(head, tail), list_min(head, tail))
}

theorem le_list_min_iff_list_lower_bound[T: LinearOrder](lower: T, head: T, tail: List[T]) {
    lower <= list_min(head, tail) = list_lower_bound(List.cons(head, tail), lower)
}

/// The minimum of a non-empty list belongs to the list.
theorem list_min_contains[T: LinearOrder](head: T, tail: List[T]) {
    List.cons(head, tail).contains(list_min(head, tail))
}

theorem list_min_lte_contains[T: LinearOrder](head: T, tail: List[T], item: T) {
    List.cons(head, tail).contains(item) implies list_min(head, tail) <= item
}

/// A list containing an element has a least element.
theorem list_contains_has_least[T: LinearOrder](items: List[T], witness: T) {
    items.contains(witness) implies exists(least: T) {
        items.contains(least) and forall(item: T) {
            items.contains(item) implies least <= item
        }
    }
}

theorem list_min_lte_head[T: LinearOrder](head: T, tail: List[T]) {
    list_min(head, tail) <= head
}

theorem list_min_add_cons[T: LinearOrder](head: T, tail: List[T], next: T, rest: List[T]) {
    list_min(head, tail + List.cons(next, rest)) =
    list_min(head, tail).min(list_min(next, rest))
}

theorem list_max_add_cons[T: LinearOrder](head: T, tail: List[T], next: T, rest: List[T]) {
    list_max(head, tail + List.cons(next, rest)) =
    list_max(head, tail).max(list_max(next, rest))
}

theorem list_min_add_cons_lte_left[T: LinearOrder](head: T, tail: List[T], next: T, rest: List[T]) {
    list_min(head, tail + List.cons(next, rest)) <= list_min(head, tail)
}

theorem list_min_add_cons_lte_right[T: LinearOrder](head: T, tail: List[T], next: T, rest: List[T]) {
    list_min(head, tail + List.cons(next, rest)) <= list_min(next, rest)
}

theorem list_max_lte_add_cons_left[T: LinearOrder](head: T, tail: List[T], next: T, rest: List[T]) {
    list_max(head, tail) <= list_max(head, tail + List.cons(next, rest))
}

theorem list_max_lte_add_cons_right[T: LinearOrder](head: T, tail: List[T], next: T, rest: List[T]) {
    list_max(next, rest) <= list_max(head, tail + List.cons(next, rest))
}

theorem list_min_append[T: LinearOrder](head: T, tail: List[T], last: T) {
    list_min(head, tail.append(last)) = list_min(head, tail).min(last)
}

theorem list_max_append[T: LinearOrder](head: T, tail: List[T], last: T) {
    list_max(head, tail.append(last)) = list_max(head, tail).max(last)
}

theorem list_min_append_lte_list_min[T: LinearOrder](head: T, tail: List[T], last: T) {
    list_min(head, tail.append(last)) <= list_min(head, tail)
}

theorem list_min_append_lte_last[T: LinearOrder](head: T, tail: List[T], last: T) {
    list_min(head, tail.append(last)) <= last
}

theorem list_max_lte_append_max[T: LinearOrder](head: T, tail: List[T], last: T) {
    list_max(head, tail) <= list_max(head, tail.append(last))
}

theorem last_lte_list_max_append[T: LinearOrder](head: T, tail: List[T], last: T) {
    last <= list_max(head, tail.append(last))
}

theorem list_has_lower_bound[T: LinearOrder](head: T, tail: List[T]) {
    exists(lb: T) {
        is_lower_bound(List.cons(head, tail), lb)
    }
}

theorem list_min_max_interval_bound[T: LinearOrder](head: T, tail: List[T]) {
    is_interval_bound(List.cons(head, tail), list_min(head, tail), list_max(head, tail))
}

theorem list_min_max_contains_bounded_by_interval[T: LinearOrder](head: T, tail: List[T]) {
    lib(order).is_bounded_by_interval(List.cons(head, tail).contains,
        list_min(head, tail), list_max(head, tail))
}

theorem list_min_max_contains_bounded[T: LinearOrder](head: T, tail: List[T]) {
    lib(order).is_bounded(List.cons(head, tail).contains)
}

theorem list_min_contains_lower_bound[T: LinearOrder](head: T, tail: List[T]) {
    lib(order).is_lower_bound(List.cons(head, tail).contains, list_min(head, tail))
}

theorem list_min_contains_bounded_below[T: LinearOrder](head: T, tail: List[T]) {
    lib(order).is_bounded_below(List.cons(head, tail).contains)
}

theorem list_max_contains_upper_bound[T: LinearOrder](head: T, tail: List[T]) {
    lib(order).is_upper_bound(List.cons(head, tail).contains, list_max(head, tail))
}

theorem list_max_contains_bounded_above[T: LinearOrder](head: T, tail: List[T]) {
    lib(order).is_bounded_above(List.cons(head, tail).contains)
}

theorem list_contains_closed_interval_min_max[T: LinearOrder](head: T, tail: List[T], item: T) {
    List.cons(head, tail).contains(item) implies
    closed_interval(list_min(head, tail), list_max(head, tail), item)
}

theorem list_has_interval_bound[T: LinearOrder](head: T, tail: List[T]) {
    exists(lower: T, upper: T) {
        is_interval_bound(List.cons(head, tail), lower, upper)
    }
}

// list_permutation.ac
attributes List[T] {
    /// The list with the first occurrence of the given element removed.
    /// If the element is not present, the list is unchanged.
    define remove_one(self, item: T) -> List[T] {
        match self {
            List.nil {
                List.nil[T]
            }
            List.cons(head, tail) {
                if head = item {
                    tail
                } else {
                    List.cons(head, tail.remove_one(item))
                }
            }
        }
    }
}

theorem remove_one_cons_eq[T](head: T, tail: List[T]) {
    List.cons(head, tail).remove_one(head) = tail
}

theorem remove_one_cons_neq[T](head: T, tail: List[T], item: T) {
    head != item implies List.cons(head, tail).remove_one(item) = List.cons(head, tail.remove_one(item))
}

theorem remove_one_not_contains[T](list: List[T], item: T) {
    not list.contains(item) implies list.remove_one(item) = list
}

theorem remove_one_count_self[T](list: List[T], item: T) {
    list.contains(item) implies list.remove_one(item).count(item) + Nat.1 = list.count(item)
}

theorem remove_one_count_other[T](list: List[T], item: T, other: T) {
    item != other implies list.remove_one(item).count(other) = list.count(other)
}

/// True if two lists contain the same elements with the same multiplicities.
define is_permutation[T](a: List[T], b: List[T]) -> Bool {
    forall(x: T) { a.count(x) = b.count(x) }
}

theorem nil_count_zero[T](item: T) {
    List.nil[T].count(item) = Nat.0
}

theorem permutation_preserves_length[T](a: List[T], b: List[T]) {
    is_permutation(a, b) implies a.length = b.length
}

theorem product_remove_one[A: CommMonoid](list: List[A], item: A) {
    list.contains(item) implies item * product[A](list.remove_one(item)) = product[A](list)
}

/// Removing one source occurrence pulls its image out of a mapped product.
theorem product_map_remove_one[T, A: CommMonoid](
    list: List[T], item: T, f: T -> A
) {
    list.contains(item) implies
        f(item) * product[A](map(list.remove_one(item), f)) =
            product[A](map(list, f))
}

theorem permutation_preserves_product[A: CommMonoid](a: List[A], b: List[A]) {
    is_permutation(a, b) implies product[A](a) = product[A](b)
}

/// Mapping a permutation into a commutative monoid preserves its product.
theorem permutation_preserves_mapped_product[T, A: CommMonoid](
    a: List[T], b: List[T], f: T -> A
) {
    is_permutation(a, b) implies
        product[A](map(a, f)) = product[A](map(b, f))
}

theorem count_append[T](left: List[T], right: List[T], item: T) {
    (left + right).count(item) = left.count(item) + right.count(item)
}

theorem product_append[A: CommMonoid](left: List[A], right: List[A]) {
    product[A](left + right) = product[A](left) * product[A](right)
}

theorem unique_same_contains_imp_permutation[T](a: List[T], b: List[T]) {
    a.is_unique and b.is_unique and (forall(x: T) { a.contains(x) = b.contains(x) })
        implies is_permutation(a, b)
}

/// Filtering same-membership unique lists gives permutations.
theorem unique_same_contains_filter_imp_permutation[T](a: List[T], b: List[T], pred: T -> Bool) {
    a.is_unique and b.is_unique and
    (forall(x: T) { a.contains(x) = b.contains(x) })
        implies is_permutation(a.filter(pred), b.filter(pred))
}

/// A unique Nat list of length `n` whose entries are below `n` contains exactly `n.range`.
theorem unique_nat_list_contained_by_range_length_contains_iff(items: List[Nat], n: Nat, x: Nat) {
    items.is_unique and items.length = n and
    (forall(y: Nat) { items.contains(y) implies y < n })
        implies items.contains(x) = (x < n)
}

/// A unique Nat list of length `n` whose elements are all below `n` enumerates `n.range`.
theorem unique_nat_list_contained_by_range_length_imp_permutation(items: List[Nat], n: Nat) {
    items.is_unique and items.length = n and
    (forall(x: Nat) { items.contains(x) implies x < n })
        implies is_permutation(items, n.range)
}

/// A locally injective map from `n.range` into `n.range` permutes `n.range`.
theorem map_range_locally_injective_bounded_is_permutation(n: Nat, f: Nat -> Nat) {
    (forall(x: Nat, y: Nat) { x < n and y < n and f(x) = f(y) implies x = y }) and
    (forall(x: Nat) { x < n implies f(x) < n })
        implies is_permutation(map(n.range, f), n.range)
}

theorem sum_map_remove_one[T, A: AddCommMonoid](list: List[T], item: T, f: T -> A) {
    list.contains(item) implies f(item) + sum[A](map(list.remove_one(item), f)) = sum[A](map(list, f))
}

theorem remove_one_unique[T](list: List[T], item: T) {
    list.is_unique implies list.remove_one(item).is_unique
}

theorem remove_one_unique_not_contains_self[T](list: List[T], item: T) {
    list.is_unique implies not list.remove_one(item).contains(item)
}

theorem remove_one_contains_other[T](list: List[T], item: T, x: T) {
    x != item implies list.contains(x) = list.remove_one(item).contains(x)
}

theorem unique_same_contains_map_sum_eq[T, A: AddCommMonoid](a: List[T], b: List[T], f: T -> A) {
    a.is_unique and b.is_unique and (forall(x: T) { a.contains(x) = b.contains(x) })
        implies sum[A](map(a, f)) = sum[A](map(b, f))
}

// list_functional.ac
// This file contains functional-programming-style things for lists.

define fold_left[T, U](list: List[T], f: (U, T) -> U, init: U) -> U {
    match list {
        List.nil {
            init
        }
        List.cons(head, tail) {
            fold_left(tail, f, f(init, head))
        }
    }
}

define fold_right[T, U](list: List[T], f: (T, U) -> U, init: U) -> U {
    match list {
        List.nil {
            init
        }
        List.cons(head, tail) {
            f(head, fold_right(tail, f, init))
        }
    }
}

/// Reverses a list.
define reverse[T](list: List[T]) -> List[T] {
    match list {
        List.nil {
            List.nil[T]
        }
        List.cons(head, tail) {
            reverse(tail).append(head)
        }
    }
}

theorem reverse_length[T](list: List[T]) {
    reverse(list).length = list.length
}

theorem reverse_involution[T](list: List[T]) {
    reverse(reverse(list)) = list
}

theorem reverse_get_idx[T](list: List[T], idx: Nat) {
    idx < list.length implies reverse(list).get_idx(idx) = list.get_idx(list.length - 1 - idx)
}

theorem sum_reverse[A: AddCommMonoid](list: List[A]) {
    sum(reverse(list)) = sum(list)
}

define descending_from(n: Nat, i: Nat) -> Nat {
    n - 1 - i
}

let pick_any[T: Inhabited](list: List[T]) -> item: T satisfy {
    if list.length > 0 {
        list.contains(item)
    }
}

attributes List[T: Inhabited] {
    let pick_any: List[T] -> T = pick_any
}

theorem permutation_preserves_mapped_sum[T, A: AddCommMonoid](
    a: List[T], b: List[T], f: T -> A
) {
    is_permutation(a, b) implies
        sum[A](map(a, f)) = sum[A](map(b, f))
}
