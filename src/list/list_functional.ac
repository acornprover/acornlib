from nat import Nat, add_one_right, lt_imp_lte_suc, not_lt_zero, sub_comm,
    sub_one_sub_lt
from list.list_sum import List, map, sum
from algebra.add_comm_monoid import AddCommMonoid

numerals Nat

// This file contains functional-programming-style things for lists.

define fold_left[T, U](list: List[T], f: (U, T) -> U, init: U) -> U {
    match list {
        List.nil {
            init
        }
        List.cons(head, tail) {
            fold_left(tail, f, f(init, head))
        }
    }
}

define fold_right[T, U](list: List[T], f: (T, U) -> U, init: U) -> U {
    match list {
        List.nil {
            init
        }
        List.cons(head, tail) {
            f(head, fold_right(tail, f, init))
        }
    }
}

/// Reverses a list.
define reverse[T](list: List[T]) -> List[T] {
    match list {
        List.nil {
            List.nil[T]
        }
        List.cons(head, tail) {
            reverse(tail).append(head)
        }
    }
}

theorem reverse_length[T](list: List[T]) {
    reverse(list).length = list.length
} by {
    define p(l: List[T]) -> Bool {
        reverse(l).length = l.length
    }

    // Base case: empty list
    p(List.nil[T])

    // Induction
    forall(head: T, tail: List[T]) {
        if p(tail) {
            // Induction hypothesis: reverse(tail).length = tail.length
            p(tail) implies reverse[T](tail).length = tail.length
            // Use add_length theorem
            (reverse(tail) + List.singleton(head)).length = reverse(tail).length + List.singleton(head).length
            reverse[T](tail) + List.singleton(head) = reverse[T](tail).append(head)
            reverse[T](tail).length + List.singleton(head).length = List.singleton(head).length + reverse[T](tail).length
            List.cons(head, List.nil[T]) = List.singleton(head)
            Nat.1 + tail.length = tail.length.suc
            List.cons(head, List.nil[T]).length = List.nil[T].length.suc
            List.cons(head, tail).length = tail.length.suc
            reverse[T](tail).append(head) = reverse[T](List.cons(head, tail))
            p(List.cons(head, tail))
        }
    }

    forall(l: List[T]) {
        p(l)
    }
}

theorem reverse_add[T](list1: List[T], list2: List[T]) {
    reverse(list1 + list2) = reverse(list2) + reverse(list1)
} by {
    define p(l1: List[T]) -> Bool {
        reverse(l1 + list2) = reverse(list2) + reverse(l1)
    }

    // Base case: empty list
    p(List.nil[T])

    // Induction
    forall(head: T, tail: List[T]) {
        if p(tail) {
            // Induction hypothesis: reverse(tail + list2) = reverse(list2) + reverse(tail)
            reverse(tail + list2).append(head) = reverse(List.cons(head, tail + list2))
            reverse(list2) + reverse(tail) = reverse(tail + list2)
            reverse(list2) + (reverse(tail) + List.singleton(head)) = reverse(list2) + reverse(tail) + List.singleton(head)
            reverse(List.cons(head, tail) + list2) = reverse(list2) + reverse(List.cons(head, tail))
            p(List.cons(head, tail))
        }
    }

    forall(l: List[T]) {
        p(l)
    }
}

theorem reverse_involution[T](list: List[T]) {
    reverse(reverse(list)) = list
} by {
    define p(l: List[T]) -> Bool {
        reverse(reverse(l)) = l
    }

    // Base case: empty list
    p(List.nil[T])

    // Induction
    forall(head: T, tail: List[T]) {
        if p(tail) {
            // Induction hypothesis: reverse(reverse(tail)) = tail
            reverse(List.singleton(head)) + reverse(reverse(tail)) = List.singleton(head) + tail
            p(List.cons(head, tail))
        }
    }

    forall(l: List[T]) {
        p(l)
    }
}

theorem reverse_contains[T](list: List[T], item: T) {
    reverse(list).contains(item) = list.contains(item)
} by {
    define p(l: List[T]) -> Bool {
        reverse(l).contains(item) = l.contains(item)
    }

    // Base case: empty list
    p(List.nil[T])

    // Induction
    forall(head: T, tail: List[T]) {
        if p(tail) {
            // Induction hypothesis: reverse(tail).contains(item) = tail.contains(item)
            reverse(List.cons(head, tail)).contains(item) = (reverse(tail) + List.singleton(head)).contains(item)

            // Show both directions of the equivalence
            if (reverse(tail) + List.singleton(head)).contains(item) {
                p(tail) implies reverse[T](tail).contains(item) = tail.contains(item)
                (reverse[T](tail) + List.cons(head, List.nil[T])).contains(item) implies reverse[T](tail).contains(item) or List.cons(head, List.nil[T]).contains(item)
                List.cons(head, List.nil[T]) = List.singleton(head)
                List.cons(head, List.nil[T]).contains(item) implies (List.cons(head, List.nil[T]) + tail).contains(item)
                List.nil[T] + tail = tail
                List.cons(head, List.nil[T]) + tail = List.cons(head, List.nil[T] + tail)
                not tail.contains(item) or List.cons(head, tail).contains(item) or head = item
                if reverse(tail).contains(item) {
                } else {
                }
                List.cons(head, tail).contains(item)
            }

            if List.cons(head, tail).contains(item) {
                if head = item {
                    List.singleton(head).contains(item)
                    (reverse(tail) + List.singleton(head)).contains(item)
                } else {
                    reverse(tail).contains(item)
                    (reverse(tail) + List.singleton(head)).contains(item)
                }
                (reverse(tail) + List.singleton(head)).contains(item)
            }

            // Both directions established, so equality holds
            (reverse(tail) + List.singleton(head)).contains(item) = List.cons(head, tail).contains(item)
            p(List.cons(head, tail))
        }
    }

    forall(l: List[T]) {
        p(l)
    }
}

/// The element at index idx in a reversed list is the element at index (length - 1 - idx) in the original list.
theorem reverse_get_idx[T](list: List[T], idx: Nat) {
    idx < list.length implies reverse(list).get_idx(idx) = list.get_idx(list.length - 1 - idx)
} by {
    define p(l: List[T]) -> Bool {
        idx < l.length implies reverse(l).get_idx(idx) = l.get_idx(l.length - 1 - idx)
    }

    // Base case: empty list has length 0, so idx < 0 is false
    p(List.nil[T])

    // Induction
    forall(head: T, tail: List[T]) {
        if p(tail) {
            if idx < List.cons(head, tail).length {

                if idx < tail.length {
                    // Case 1: idx is in the reversed tail part
                    idx < reverse(tail).length

                    // Show tail.get_idx(tail.length - 1 - idx) = List.cons(head, tail).get_idx(tail.length.suc - 1 - idx)
                    // We have tail.length.suc - 1 - idx = (tail.length + 1) - 1 - idx = tail.length - idx
                    // And tail.length - idx > 0 (since idx < tail.length)
                    // So List.cons(head, tail).get_idx(tail.length - idx) = tail.get_idx(tail.length - idx - 1)
                    // And we need to show tail.length - idx - 1 = tail.length - 1 - idx

                    // Use sub_comm: (tail.length - idx) - 1 = (tail.length - 1) - idx
                    tail.length.suc - 1 - idx = tail.length - idx
                    tail.length - idx > 0
                    List.cons(head, tail).get_idx(tail.length - idx) = tail.get_idx(tail.length - idx - 1)

                    tail.get_idx(tail.length - idx - 1) = tail.get_idx(tail.length - 1 - idx)
                    List.cons(head, tail).get_idx(tail.length.suc - 1 - idx) = tail.get_idx(tail.length - 1 - idx)

                    // Connect to the induction hypothesis
                    reverse(tail).get_idx(idx) = tail.get_idx(tail.length - 1 - idx)

                    // The append doesn't affect indices < reverse(tail).length
                    (reverse(tail) + List.singleton(head)).get_idx(idx) = reverse(tail).get_idx(idx)

                    // Show that length expressions are equal
                    List.cons(head, tail).length - 1 - idx = tail.length.suc - 1 - idx
                    List.cons(head, tail).get_idx(List.cons(head, tail).length - 1 - idx) = tail.get_idx(tail.length - 1 - idx)

                    reverse(List.cons(head, tail)).get_idx(idx) = List.cons(head, tail).get_idx(List.cons(head, tail).length - 1 - idx)
                    p(List.cons(head, tail))
                } else {
                    // Case 2: idx = tail.length
                    (reverse(tail) + List.singleton(head)).get_idx(tail.length) = Option.some(head)
                    List.cons(head, tail).get_idx(0) = Option.some(head)

                    // Show tail.length.suc - 1 - tail.length = 0
                    tail.length - tail.length = 0

                    reverse(List.cons(head, tail)).get_idx(idx) = List.cons(head, tail).get_idx(List.cons(head, tail).length - 1 - idx)
                    p(List.cons(head, tail))
                }

                p(List.cons(head, tail))
            }
            p(List.cons(head, tail))
        }
    }

    forall(l: List[T]) {
        p(l)
    }
}

define descending_from(n: Nat, i: Nat) -> Nat {
    n - 1 - i
}

/// Helper lemma: reverse(n.range) and map(n.range, {n-1-i}) agree at every index.
theorem range_reverse_map_get_idx(n: Nat, idx: Nat) {
    idx < n implies reverse(n.range).get_idx(idx) = map(n.range, descending_from(n)).get_idx(idx)
} by {
    if idx < n {
        if n = 0 {
            // idx < 0 is impossible, so this case is vacuous
            false
        } else {
            // n > 0 case
            n > 0

            // Apply reverse_get_idx theorem
            n.range.length = n
            idx < n.range.length
            reverse(n.range).get_idx(idx) = n.range.get_idx(n.range.length - 1 - idx)
            reverse(n.range).get_idx(idx) = n.range.get_idx(n - 1 - idx)

            // Use sub_one_sub_lt to show n - 1 - idx < n
            n - 1 - idx < n

            // Apply range_idx_eq_idx theorem
            n.range.get_idx(n - 1 - idx) = Option.some(n - 1 - idx)

            // Apply map_range theorem
            map(n.range, descending_from(n)).get_idx(idx) = Option.some((n - 1) - idx)

            // Both sides equal Option.some(n - 1 - idx)
            Option.some(n - 1 - idx) = Option.some((n - 1) - idx)
            reverse(n.range).get_idx(idx) = map(n.range, descending_from(n)).get_idx(idx)
        }
        reverse(n.range).get_idx(idx) = map(n.range, descending_from(n)).get_idx(idx)
    }
}

/// The reverse of n.range equals mapping the subtraction function over n.range.
/// In other words, [0, 1, 2, ..., n-1] reversed equals [n-1, n-2, ..., 1, 0],
/// which is the same as mapping i to (n-1-i) over the original range.
theorem range_reverse_map(n: Nat) {
    reverse(n.range) = map(n.range, descending_from(n))
} by {
    let left = reverse(n.range)

    // Both lists have the same length
    n.range.length = n
    left.length = n.range.length
    left.length = map(n.range, descending_from(n)).length

    // They agree at every index by the helper lemma
    forall(idx: Nat) {
        if idx < left.length {
            idx < n
            left.get_idx(idx) = map(n.range, descending_from(n)).get_idx(idx)
        }
    }

    // Apply list extensionality to conclude the lists are equal
    left.length = map(n.range, descending_from(n)).length and (forall(idx: Nat) { idx < left.length implies left.get_idx(idx) = map(n.range, descending_from(n)).get_idx(idx) })
    left = map(n.range, descending_from(n))
}

/// The sum is invariant under list reversal.
theorem sum_reverse[A: AddCommMonoid](list: List[A]) {
    sum(reverse(list)) = sum(list)
} by {
    define p(l: List[A]) -> Bool {
        sum(reverse(l)) = sum(l)
    }

    // Base case: empty list
    p(List.nil[A])

    // Inductive step
    forall(head: A, tail: List[A]) {
        if p(tail) {
            // Induction hypothesis: sum(reverse(tail)) = sum(tail)
            p(tail) implies sum[A](reverse[A](tail)) = sum[A](tail)
            sum[A](reverse[A](tail).append(head)) = sum[A](reverse[A](tail)) + head
            p(List.cons(head, tail))
        }
    }

    forall(l: List[A]) {
        p(l)
    }
}

from data.basic.functions import Inhabited

let pick_any[T: Inhabited](list: List[T]) -> item: T satisfy {
    if list.length > 0 {
        list.contains(item)
    }
} by {
    match list {
        List.nil {
            let x: T satisfy {
                true
            }
            if list.length > 0 {
                false
            }
        }
        List.cons(head, tail) {
            if list.length > 0 {
                list.contains(head)
            }
        }
    }
}

attributes List[T: Inhabited] {
    let pick_any: List[T] -> T = pick_any
} 
