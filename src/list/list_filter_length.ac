from data.basic.functions import identity_fn
from list.list_base import List, add_length, filter_contained_by_and,
    filter_equivalent_to_and, map, map_identity, unique_implies_tail_unique, unique_length
from list.list_permutation import is_permutation, permutation_preserves_length,
    unique_same_contains_imp_permutation
from nat import Nat

/// Filtering after mapping has the same length as filtering before mapping when predicates agree pointwise.
theorem map_filter_length_of_pointwise[T, U](items: List[T], f: T -> U, p: U -> Bool, q: T -> Bool) {
    forall(x: T) { items.contains(x) implies p(f(x)) = q(x) } implies
    map[T, U](items, f).filter(p).length = items.filter(q).length
} by {
    define r(xs: List[T]) -> Bool {
        forall(x: T) { xs.contains(x) implies p(f(x)) = q(x) } implies
        map[T, U](xs, f).filter(p).length = xs.filter(q).length
    }
    if forall(x: T) { List.nil[T].contains(x) implies p(f(x)) = q(x) } {
        map[T, U](List.nil[T], f).filter(p) = List.nil[U]
        List.nil[T].filter(q).length = Nat.0
        map[T, U](List.nil[T], f).filter(p).length = List.nil[T].filter(q).length
    }
    r(List.nil[T])
    forall(head: T, tail: List[T]) {
        if r(tail) {
            if forall(x: T) { List.cons[T](head, tail).contains(x) implies p(f(x)) = q(x) } {
                forall(x: T) {
                    if tail.contains(x) {
                        List.cons[T](head, tail).contains(x)
                        p(f(x)) = q(x)
                    }
                }
                r(tail)
                map[T, U](tail, f).filter(p).length = tail.filter(q).length
                if q(head) {
                    List.cons[U](f(head), map[T, U](tail, f)).filter(p) =
                        List.cons[U](f(head), map[T, U](tail, f).filter(p))
                    List.cons[T](head, tail).filter(q) = List.cons[T](head, tail.filter(q))
                    map[T, U](List.cons[T](head, tail), f).filter(p).length =
                        map[T, U](tail, f).filter(p).length.suc
                    List.cons[T](head, tail).filter(q).length = tail.filter(q).length.suc
                    map[T, U](List.cons[T](head, tail), f).filter(p).length =
                        List.cons[T](head, tail).filter(q).length
                }
                if not q(head) {
                    List.cons[U](f(head), map[T, U](tail, f)).filter(p) =
                        map[T, U](tail, f).filter(p)
                    List.cons[T](head, tail).filter(q) = tail.filter(q)
                    map[T, U](List.cons[T](head, tail), f).filter(p).length =
                        List.cons[T](head, tail).filter(q).length
                }
                map[T, U](List.cons[T](head, tail), f).filter(p).length =
                    List.cons[T](head, tail).filter(q).length
            }
            r(List.cons[T](head, tail))
        }
    }
    List.induction(function(xs: List[T]) { r(xs) })
    forall(xs: List[T]) { r(xs) }
    r(items)
}

/// Filters with pointwise equal predicates have equal lengths.
theorem filter_length_of_pointwise[T](items: List[T], p: T -> Bool, q: T -> Bool) {
    forall(x: T) { items.contains(x) implies p(x) = q(x) } implies
        items.filter(p).length = items.filter(q).length
} by {
    if forall(x: T) { items.contains(x) implies p(x) = q(x) } {
        forall(x: T) {
            if items.contains(x) {
                p(identity_fn[T](x)) = q(x)
            }
        }
        map_filter_length_of_pointwise[T, T](items, identity_fn[T], p, q)
        map[T, T](items, identity_fn[T]).filter(p).length = items.filter(q).length
        map_identity(items)
        items.filter(p).length = items.filter(q).length
    }
}

/// Filtering by a false predicate produces an empty-length list.
theorem filter_false_length[T](items: List[T]) {
    items.filter(function(x: T) { false }).length = Nat.0
} by {
    define p(xs: List[T]) -> Bool {
        xs.filter(function(x: T) { false }).length = Nat.0
    }
    p(List.nil[T])
    forall(head: T, tail: List[T]) {
        if p(tail) {
            List.cons[T](head, tail).filter(function(x: T) { false }) =
                tail.filter(function(x: T) { false })
            p(List.cons[T](head, tail))
        }
    }
    List.induction(function(xs: List[T]) { p(xs) })
    forall(xs: List[T]) { p(xs) }
}

lemma unique_cons_not_contains[T](head: T, tail: List[T]) {
    List.cons(head, tail).is_unique implies not tail.contains(head)
} by {
    if List.cons(head, tail).is_unique {
        if tail.contains(head) {
            let cons_list = List.cons(head, tail)
            List.cons(head, tail).unique = List.cons(head, tail)
            tail.unique = List.cons(head, tail)
            unique_length(tail)
            List.cons(head, tail).length = tail.length.suc
            tail.length.suc <= tail.length
            false
        }
    }
}

theorem cons_unique_of_tail_unique_not_contains[T](head: T, tail: List[T]) {
    tail.is_unique and not tail.contains(head) implies List.cons(head, tail).is_unique
} by {
    if tail.is_unique and not tail.contains(head) {
        tail.unique = tail
        List.cons(head, tail).is_unique
    }
}

/// Filtering distributes over list concatenation.
theorem filter_add[T](left: List[T], right: List[T], pred: T -> Bool) {
    (left + right).filter(pred) = left.filter(pred) + right.filter(pred)
} by {
    define p(xs: List[T]) -> Bool {
        (xs + right).filter(pred) = xs.filter(pred) + right.filter(pred)
    }

    p(List.nil[T])
    forall(head: T, tail: List[T]) {
        if p(tail) {
            if pred(head) {
                (List.cons(head, tail) + right).filter(pred) =
                    List.cons(head, (tail + right).filter(pred))
                List.cons(head, tail).filter(pred) = List.cons(head, tail.filter(pred))
                (tail + right).filter(pred) = tail.filter(pred) + right.filter(pred)
                (List.cons(head, tail).filter(pred) + right.filter(pred)) =
                    List.cons(head, tail.filter(pred) + right.filter(pred))
                (List.cons(head, tail) + right).filter(pred) =
                    List.cons(head, tail).filter(pred) + right.filter(pred)
            }
            if not pred(head) {
                (List.cons(head, tail) + right).filter(pred) = (tail + right).filter(pred)
                List.cons(head, tail).filter(pred) = tail.filter(pred)
                (tail + right).filter(pred) = tail.filter(pred) + right.filter(pred)
                (List.cons(head, tail) + right).filter(pred) =
                    List.cons(head, tail).filter(pred) + right.filter(pred)
            }
            p(List.cons(head, tail))
        }
    }

    p(left)
}

/// The length of a filtered concatenation is the sum of filtered lengths.
theorem filter_add_length[T](left: List[T], right: List[T], pred: T -> Bool) {
    (left + right).filter(pred).length = left.filter(pred).length + right.filter(pred).length
} by {
    filter_add(left, right, pred)
    add_length(left.filter(pred), right.filter(pred))
}

/// Filtering a unique list preserves uniqueness.
theorem filter_preserves_unique[T](list: List[T], pred: T -> Bool) {
    list.is_unique implies list.filter(pred).is_unique
} by {
    define p(xs: List[T]) -> Bool {
        xs.is_unique implies xs.filter(pred).is_unique
    }

    p(List.nil[T])
    forall(head: T, tail: List[T]) {
        if p(tail) {
            if List.cons(head, tail).is_unique {
                unique_implies_tail_unique(head, tail)
                unique_cons_not_contains(head, tail)
                tail.filter(pred).is_unique
                if pred(head) {
                    List.cons(head, tail).filter(pred) = List.cons(head, tail.filter(pred))
                    if tail.filter(pred).contains(head) {
                        filter_contained_by_and(tail, pred, head)
                        false
                    }
                    cons_unique_of_tail_unique_not_contains(head, tail.filter(pred))
                    List.cons(head, tail).filter(pred).is_unique
                }
                if not pred(head) {
                    let cons_list = List.cons(head, tail)
                    List.cons(head, tail).filter(pred) = tail.filter(pred)
                    List.cons(head, tail).filter(pred).is_unique
                }
                List.cons(head, tail).filter(pred).is_unique
            }
            if not List.cons(head, tail).is_unique {
            }
            p(List.cons(head, tail))
        }
    }

    List.induction(function(xs: List[T]) { p(xs) })
    forall(xs: List[T]) {
        p(xs)
    }
}

/// Lists with the same elements have filters with the same membership.
theorem same_contains_filter_contains_iff[T](a: List[T], b: List[T], pred: T -> Bool, x: T) {
    (forall(y: T) { a.contains(y) = b.contains(y) }) implies
    a.filter(pred).contains(x) = b.filter(pred).contains(x)
} by {
    if forall(y: T) { a.contains(y) = b.contains(y) } {
        filter_equivalent_to_and(a, pred, x)
        filter_equivalent_to_and(b, pred, x)
        a.filter(pred).contains(x) = b.filter(pred).contains(x)
    }
}

/// Filtering same-membership unique lists gives permutations.
theorem unique_same_contains_filter_imp_permutation[T](a: List[T], b: List[T], pred: T -> Bool) {
    a.is_unique and b.is_unique and
    (forall(x: T) { a.contains(x) = b.contains(x) })
        implies is_permutation(a.filter(pred), b.filter(pred))
} by {
    if a.is_unique and b.is_unique and
        (forall(x: T) { a.contains(x) = b.contains(x) }) {
        filter_preserves_unique(a, pred)
        filter_preserves_unique(b, pred)
        a.filter(pred).is_unique
        b.filter(pred).is_unique
        forall(x: T) {
            same_contains_filter_contains_iff(a, b, pred, x)
            a.filter(pred).contains(x) = b.filter(pred).contains(x)
        }
        unique_same_contains_imp_permutation(a.filter(pred), b.filter(pred))
        is_permutation(a.filter(pred), b.filter(pred))
    }
}

/// Unique lists with the same elements have equal filtered lengths.
theorem unique_same_contains_filter_length[T](a: List[T], b: List[T], pred: T -> Bool) {
    a.is_unique and b.is_unique and
    (forall(x: T) { a.contains(x) = b.contains(x) })
        implies a.filter(pred).length = b.filter(pred).length
} by {
    if a.is_unique and b.is_unique and
        (forall(x: T) { a.contains(x) = b.contains(x) }) {
        filter_preserves_unique(a, pred)
        filter_preserves_unique(b, pred)
        a.filter(pred).is_unique
        b.filter(pred).is_unique
        forall(x: T) {
            filter_equivalent_to_and(a, pred, x)
            filter_equivalent_to_and(b, pred, x)
            a.filter(pred).contains(x) = b.filter(pred).contains(x)
        }
        unique_same_contains_imp_permutation(a.filter(pred), b.filter(pred))
        permutation_preserves_length(a.filter(pred), b.filter(pred))
        a.filter(pred).length = b.filter(pred).length
    }
}
