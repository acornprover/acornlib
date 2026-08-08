from list.list_base import List
from algebra.comm_monoid import CommMonoid

/// Computes the product of all elements in a list (requires elements to form
/// a commutative monoid). The product of the empty list is the monoid's
/// identity element.
define product[A: CommMonoid](items: List[A]) -> A {
    match items {
        List.nil {
            A.1
        }
        List.cons(head, tail) {
            head * product(tail)
        }
    }
}

/// The product of the empty list is the multiplicative identity.
theorem product_nil[A: CommMonoid] {
    product(List.nil[A]) = A.1
}

/// The product of a cons list factors as head times the product of the tail.
theorem product_cons[A: CommMonoid](head: A, tail: List[A]) {
    product(List.cons(head, tail)) = head * product(tail)
}
