from list.list_base import List
from order import LinearOrder, PartialOrder, lte_antisymm, lte_trans,
    lte_min_of_bounds, max_lte_of_upper_bounds,
    min_assoc_rev, max_assoc_rev, min_is_one, max_is_one, min_lte_left, min_lte_right,
    lte_max_left, lte_max_right
from order import closed_interval
from list.list_lattice import list_lower_bound, list_upper_bound,
    list_lower_bound_nil, list_upper_bound_nil,
    list_lower_bound_cons_iff, list_upper_bound_cons_iff,
    list_lower_bound_add_iff, list_upper_bound_add_iff,
    list_lower_bound_singleton_iff, list_upper_bound_singleton_iff,
    list_lower_bound_append_iff, list_upper_bound_append_iff

/// Theorems about lists of ordered types.

/// True if t is an upper bound for all items in the list.
/// An upper bound t satisfies: every element in the list is ≤ t.
define is_upper_bound[T: LinearOrder](list: List[T], t: T) -> Bool {
    match list {
        List.nil {
            true
        }
        List.cons(head, tail) {
            head <= t and is_upper_bound(tail, t)
        }
    }
}

/// If ub is an upper bound for a list and the list contains c, then c ≤ ub.
theorem upper_bound_contains[T: LinearOrder](list: List[T], ub: T, c: T) {
    is_upper_bound(list, ub) and list.contains(c) implies c <= ub
} by {
    define p(l: List[T]) -> Bool {
        is_upper_bound(l, ub) and l.contains(c) implies c <= ub
    }

    // Base case: empty list
    not List.nil[T].contains(c)
    p(List.nil[T])

    // Induction
    forall(head: T, tail: List[T]) {
        if p(tail) {
            if is_upper_bound(List.cons(head, tail), ub) and List.cons(head, tail).contains(c) {
                // is_upper_bound means head <= ub and tail is also bounded
                is_upper_bound(tail, ub)

                // contains means c is either head or in tail
                if head = c {
                    c <= ub
                } else {
                    c <= ub
                }
            }
            p(List.cons(head, tail))
        }
    }

}

/// If ub is an upper bound for (list1 + list2), then ub is an upper bound for list1.
theorem upper_bound_add_left[T: LinearOrder](list1: List[T], list2: List[T], ub: T) {
    is_upper_bound(list1 + list2, ub) implies is_upper_bound(list1, ub)
} by {
    define p(l1: List[T]) -> Bool {
        is_upper_bound(l1 + list2, ub) implies is_upper_bound(l1, ub)
    }

    // Base case: empty list
    is_upper_bound(List.nil[T], ub)
    p(List.nil[T])

    // Induction
    forall(head: T, tail: List[T]) {
        if p(tail) {
            p(List.cons(head, tail))
        }
    }

}

/// If ub is an upper bound for (list1 + list2), then ub is an upper bound for list2.
theorem upper_bound_add_right[T: LinearOrder](list1: List[T], list2: List[T], ub: T) {
    is_upper_bound(list1 + list2, ub) implies is_upper_bound(list2, ub)
} by {
    define p(l1: List[T]) -> Bool {
        is_upper_bound(l1 + list2, ub) implies is_upper_bound(list2, ub)
    }

    // Base case: empty list
    p(List.nil[T])

    // Induction
    forall(head: T, tail: List[T]) {
        if p(tail) {
            p(List.cons(head, tail))
        }
    }

}

/// If ub1 is an upper bound and ub1 ≤ ub2, then ub2 is also an upper bound.
theorem upper_bound_monotone[T: LinearOrder](list: List[T], ub1: T, ub2: T) {
    is_upper_bound(list, ub1) and ub1 <= ub2 implies is_upper_bound(list, ub2)
} by {
    define p(l: List[T]) -> Bool {
        is_upper_bound(l, ub1) and ub1 <= ub2 implies is_upper_bound(l, ub2)
    }

    // Base case: empty list
    is_upper_bound(List.nil[T], ub2)
    p(List.nil[T])

    // Induction
    forall(head: T, tail: List[T]) {
        if p(tail) {
            p(List.cons(head, tail))
        }
    }

}

/// Recursive list upper bounds are the same as membership-predicate upper bounds.
theorem upper_bound_iff_list_upper_bound[T: LinearOrder](list: List[T], upper: T) {
    is_upper_bound(list, upper) = list_upper_bound(list, upper)
} by {
    define p(l: List[T]) -> Bool {
        is_upper_bound(l, upper) = list_upper_bound(l, upper)
    }
    forall(x: T) {
        if List.nil[T].contains(x) {
            false
        }
    }
    list_upper_bound(List.nil[T], upper)
    p(List.nil[T])
    forall(head: T, tail: List[T]) {
        if p(tail) {
            if is_upper_bound(List.cons(head, tail), upper) {
                head <= upper
                is_upper_bound(tail, upper)
                list_upper_bound(tail, upper)
                forall(x: T) {
                    if List.cons(head, tail).contains(x) {
                        if head = x {
                            x <= upper
                        } else {
                            tail.contains(x)
                            x <= upper
                        }
                    }
                }
                list_upper_bound(List.cons(head, tail), upper)
            }
            if list_upper_bound(List.cons(head, tail), upper) {
                list_upper_bound(List.cons(head, tail), upper) = forall(x: T) {
                    List.cons(head, tail).contains(x) implies x <= upper
                }
                forall(x: T) {
                    if tail.contains(x) {
                        List.cons(head, tail).contains(x)
                        x <= upper
                    }
                }
                list_upper_bound(tail, upper)
                is_upper_bound(tail, upper)
                is_upper_bound(List.cons(head, tail), upper)
            }
            p(List.cons(head, tail))
        }
    }
    p(list)
}

/// Recursive list upper bounds give membership-predicate upper bounds.
theorem upper_bound_imp_list_upper_bound[T: LinearOrder](list: List[T], upper: T) {
    is_upper_bound(list, upper) implies list_upper_bound(list, upper)
} by {
    if is_upper_bound(list, upper) {
        upper_bound_iff_list_upper_bound(list, upper)
        list_upper_bound(list, upper)
    }
}

/// Membership-predicate upper bounds give recursive list upper bounds.
theorem list_upper_bound_imp_upper_bound[T: LinearOrder](list: List[T], upper: T) {
    list_upper_bound(list, upper) implies is_upper_bound(list, upper)
} by {
    if list_upper_bound(list, upper) {
        upper_bound_iff_list_upper_bound(list, upper)
        is_upper_bound(list, upper)
    }
}

/// Every point is an upper bound for the empty list.
theorem upper_bound_nil[T: LinearOrder](upper: T) {
    is_upper_bound(List.nil[T], upper)
} by {
    list_upper_bound_nil(upper)
    list_upper_bound_imp_upper_bound(List.nil[T], upper)
}

/// Upper bounds for a cons list are exactly upper bounds for the head and tail.
theorem upper_bound_cons_iff[T: LinearOrder](head: T, tail: List[T], upper: T) {
    is_upper_bound(List.cons(head, tail), upper) =
    (head <= upper and is_upper_bound(tail, upper))
} by {
    upper_bound_iff_list_upper_bound(List.cons(head, tail), upper)
    list_upper_bound_cons_iff(head, tail, upper)
    upper_bound_iff_list_upper_bound(tail, upper)
}

/// Upper bounds for a concatenation are exactly upper bounds for both lists.
theorem upper_bound_add_iff[T: LinearOrder](left: List[T], right: List[T], upper: T) {
    is_upper_bound(left + right, upper) =
    (is_upper_bound(left, upper) and is_upper_bound(right, upper))
} by {
    upper_bound_iff_list_upper_bound(left + right, upper)
    list_upper_bound_add_iff(left, right, upper)
    upper_bound_iff_list_upper_bound(left, upper)
    upper_bound_iff_list_upper_bound(right, upper)
}

/// An upper bound for a singleton list is exactly a point above the singleton element.
theorem upper_bound_singleton_iff[T: LinearOrder](item: T, upper: T) {
    is_upper_bound(List.singleton(item), upper) = (item <= upper)
} by {
    upper_bound_iff_list_upper_bound(List.singleton(item), upper)
    list_upper_bound_singleton_iff(item, upper)
}

/// Upper bounds after append are exactly upper bounds for the old list and the appended element.
theorem upper_bound_append_iff[T: LinearOrder](items: List[T], last: T, upper: T) {
    is_upper_bound(items.append(last), upper) =
    (is_upper_bound(items, upper) and last <= upper)
} by {
    upper_bound_iff_list_upper_bound(items.append(last), upper)
    list_upper_bound_append_iff(items, last, upper)
    upper_bound_iff_list_upper_bound(items, upper)
}

/// The maximum element of a non-empty list.
define list_max[T: LinearOrder](head: T, tail: List[T]) -> T {
    match tail {
        List.nil {
            head
        }
        List.cons(next, rest) {
            head.max(list_max(next, rest))
        }
    }
}

/// The maximum element of a list is an upper bound for that list.
theorem list_max_is_upper_bound[T: LinearOrder](head: T, tail: List[T]) {
    is_upper_bound(List.cons(head, tail), list_max(head, tail))
} by {
    define p(t: List[T]) -> Bool {
        forall(h: T) {
            is_upper_bound(List.cons(h, t), list_max(h, t))
        }
    }

    // Base case: tail is nil (singleton list)
    forall(h: T) {
        h <= h
        is_upper_bound(List.cons(h, List.nil[T]), list_max(h, List.nil[T]))
    }
    p(List.nil[T])

    // Inductive case
    forall(next: T, rest: List[T]) {
        if p(rest) {
            forall(h: T) {
                // By induction hypothesis for next and rest specifically

                let m = list_max(h, List.cons(next, rest))

                // We know h.max(x) >= h and h.max(x) >= x for any x

                // Since list_max(next, rest) is an upper bound for cons(next, rest),
                // and m >= list_max(next, rest), use monotonicity
                is_upper_bound(List.cons(next, rest), list_max(next, rest))
                h.max(list_max(next, rest)) = list_max(h, List.cons(next, rest))
                h.max(list_max(next, rest)) >= list_max(next, rest)
                list_max(next, rest) <= list_max(h, List.cons(next, rest))
                is_upper_bound(List.cons(next, rest), m)

                // Now we can show is_upper_bound(cons(h, cons(next, rest)), m)

                // This is exactly list_max(h, cons(next, rest))
                h.max(list_max(next, rest)) >= h
                h <= list_max(h, List.cons(next, rest))
                is_upper_bound(List.cons(h, List.cons(next, rest)), list_max(h, List.cons(next, rest)))
            }

            // We've shown forall(h: T) { is_upper_bound(List.cons(h, List.cons(next, rest)), list_max(h, List.cons(next, rest))) }
            // which is exactly p(List.cons(next, rest))
            p(List.cons(next, rest))
        }
    }

    forall(t: List[T]) {
        p(t)
    }

    // Apply to our specific head and tail
}

/// Any upper bound for every element of a non-empty list is above its maximum.
theorem list_max_lte_of_upper_bound[T: LinearOrder](head: T, tail: List[T], upper: T) {
    is_upper_bound(List.cons(head, tail), upper) implies
    list_max(head, tail) <= upper
} by {
    define p(t: List[T]) -> Bool {
        forall(h: T, bound: T) {
            is_upper_bound(List.cons(h, t), bound) implies
            list_max(h, t) <= bound
        }
    }
    forall(h: T, bound: T) {
        if is_upper_bound(List.cons(h, List.nil[T]), bound) {
            upper_bound_contains(List.cons(h, List.nil[T]), bound, h)
            list_max(h, List.nil[T]) <= bound
        }
    }
    p(List.nil[T])
    forall(next: T, rest: List[T]) {
        if p(rest) {
            forall(h: T, bound: T) {
                if is_upper_bound(List.cons(h, List.cons(next, rest)), bound) {
                    upper_bound_add_right(List.singleton(h), List.cons(next, rest), bound)
                    is_upper_bound(List.cons(next, rest), bound)
                    list_max(next, rest) <= bound
                    upper_bound_contains(List.cons(h, List.cons(next, rest)), bound, h)
                    h <= bound
                    max_lte_of_upper_bounds(h, list_max(next, rest), bound)
                    h.max(list_max(next, rest)) <= bound
                    list_max(h, List.cons(next, rest)) <= bound
                }
            }
            p(List.cons(next, rest))
        }
    }
    p(tail)
}

/// The maximum of a non-empty list is the least upper bound of its elements.
theorem list_max_lte_iff_upper_bound[T: LinearOrder](head: T, tail: List[T], upper: T) {
    list_max(head, tail) <= upper = is_upper_bound(List.cons(head, tail), upper)
} by {
    if list_max(head, tail) <= upper {
        list_max_is_upper_bound(head, tail)
        upper_bound_monotone(List.cons(head, tail), list_max(head, tail), upper)
        is_upper_bound(List.cons(head, tail), upper)
    }
    if is_upper_bound(List.cons(head, tail), upper) {
        list_max_lte_of_upper_bound(head, tail, upper)
        list_max(head, tail) <= upper
    }
}

/// The list maximum is below a point exactly when the point is an upper bound.
theorem list_max_le_iff[T: LinearOrder](head: T, tail: List[T], upper: T) {
    list_max(head, tail) <= upper = is_upper_bound(List.cons(head, tail), upper)
} by {
    list_max_lte_iff_upper_bound(head, tail, upper)
}

/// The maximum of a non-empty list is a membership-predicate upper bound.
theorem list_max_is_list_upper_bound[T: LinearOrder](head: T, tail: List[T]) {
    list_upper_bound(List.cons(head, tail), list_max(head, tail))
} by {
    list_max_is_upper_bound(head, tail)
    upper_bound_imp_list_upper_bound(List.cons(head, tail), list_max(head, tail))
}

/// The list maximum is below a point exactly when that point is a membership-predicate upper bound.
theorem list_max_le_iff_list_upper_bound[T: LinearOrder](head: T, tail: List[T], upper: T) {
    list_max(head, tail) <= upper = list_upper_bound(List.cons(head, tail), upper)
} by {
    list_max_lte_iff_upper_bound(head, tail, upper)
    upper_bound_iff_list_upper_bound(List.cons(head, tail), upper)
}

/// The maximum of a non-empty list belongs to the list.
theorem list_max_contains[T: LinearOrder](head: T, tail: List[T]) {
    List.cons(head, tail).contains(list_max(head, tail))
} by {
    define p(items: List[T]) -> Bool {
        forall(first: T) {
            List.cons(first, items).contains(list_max(first, items))
        }
    }
    forall(first: T) {
        list_max(first, List.nil[T]) = first
        List.cons(first, List.nil[T]).contains(first)
    }
    p(List.nil[T])
    forall(next: T, rest: List[T]) {
        if p(rest) {
            forall(first: T) {
                list_max(first, List.cons(next, rest)) =
                    first.max(list_max(next, rest))
                max_is_one(first, list_max(next, rest))
                if first.max(list_max(next, rest)) = first {
                    List.cons(first, List.cons(next, rest)).contains(
                        list_max(first, List.cons(next, rest)))
                } else {
                    first.max(list_max(next, rest)) = list_max(next, rest)
                    list_max(first, List.cons(next, rest)) = list_max(next, rest)
                    List.cons(next, rest).contains(list_max(next, rest))
                    List.cons(first, List.cons(next, rest)).contains(
                        list_max(first, List.cons(next, rest)))
                }
            }
            p(List.cons(next, rest))
        }
    }
    List.induction(p)
    p(tail)
}

/// Every element of a non-empty list is below its maximum.
theorem contains_lte_list_max[T: LinearOrder](head: T, tail: List[T], item: T) {
    List.cons(head, tail).contains(item) implies item <= list_max(head, tail)
} by {
    if List.cons(head, tail).contains(item) {
        list_max_is_upper_bound(head, tail)
        upper_bound_contains(List.cons(head, tail), list_max(head, tail), item)
        item <= list_max(head, tail)
    }
}

/// A list containing an element has a greatest element.
theorem list_contains_has_greatest[T: LinearOrder](items: List[T], witness: T) {
    items.contains(witness) implies exists(greatest: T) {
        items.contains(greatest) and forall(item: T) {
            items.contains(item) implies item <= greatest
        }
    }
} by {
    define p(xs: List[T]) -> Bool {
        xs.contains(witness) implies exists(greatest: T) {
            xs.contains(greatest) and forall(item: T) {
                xs.contains(item) implies item <= greatest
            }
        }
    }
    not List.nil[T].contains(witness)
    p(List.nil[T])
    forall(head: T, tail: List[T]) {
        if p(tail) {
            if List.cons(head, tail).contains(witness) {
                list_max_contains(head, tail)
                forall(item: T) {
                    if List.cons(head, tail).contains(item) {
                        contains_lte_list_max(head, tail, item)
                    }
                }
                exists(greatest: T) {
                    greatest = list_max(head, tail) and
                    List.cons(head, tail).contains(greatest) and forall(item: T) {
                        List.cons(head, tail).contains(item) implies item <= greatest
                    }
                }
            }
            p(List.cons(head, tail))
        }
    }
    List.induction(p)
    p(items)
}

/// A nonempty list in a partial order has a maximal element.
theorem list_cons_has_maximal[T: PartialOrder](candidate: T, items: List[T]) {
    exists(maximal: T) {
        List.cons(candidate, items).contains(maximal) and forall(item: T) {
            List.cons(candidate, items).contains(item) and maximal <= item implies item = maximal
        }
    }
} by {
    define p(xs: List[T]) -> Bool {
        exists(maximal: T) {
            List.cons(candidate, xs).contains(maximal) and forall(item: T) {
                List.cons(candidate, xs).contains(item) and maximal <= item implies item = maximal
            }
        }
    }
    exists(maximal: T) {
        maximal = candidate and
        List.cons(candidate, List.nil[T]).contains(maximal) and forall(item: T) {
            List.cons(candidate, List.nil[T]).contains(item) and maximal <= item
            implies item = maximal
        }
    }
    p(List.nil[T])
    forall(head: T, tail: List[T]) {
        if p(tail) {
            let maximal: T satisfy {
                List.cons(candidate, tail).contains(maximal) and forall(item: T) {
                    List.cons(candidate, tail).contains(item) and maximal <= item
                    implies item = maximal
                }
            }
            if maximal <= head {
                forall(item: T) {
                    if List.cons(candidate, List.cons(head, tail)).contains(item) and
                        head <= item {
                        if item = head {
                            item = head
                        } else {
                            if candidate = item {
                                List.cons(candidate, tail).contains(item)
                            } else {
                                List.cons(head, tail).contains(item)
                                tail.contains(item)
                                List.cons(candidate, tail).contains(item)
                            }
                            lte_trans(maximal, head, item)
                            maximal <= item
                            item = maximal
                            head <= maximal
                            lte_antisymm(maximal, head)
                            maximal = head
                            false
                        }
                    }
                }
                List.cons(head, tail).contains(head)
                List.cons(candidate, List.cons(head, tail)).contains(head)
                exists(result: T) {
                    result = head and
                    List.cons(candidate, List.cons(head, tail)).contains(result) and
                    forall(item: T) {
                        List.cons(candidate, List.cons(head, tail)).contains(item) and
                        result <= item implies item = result
                    }
                }
                p(List.cons(head, tail))
            } else {
                forall(item: T) {
                    if List.cons(candidate, List.cons(head, tail)).contains(item) and
                        maximal <= item {
                        if item = head {
                            maximal <= head
                            false
                        } else {
                            if candidate = item {
                                List.cons(candidate, tail).contains(item)
                            } else {
                                List.cons(head, tail).contains(item)
                                tail.contains(item)
                                List.cons(candidate, tail).contains(item)
                            }
                            item = maximal
                        }
                    }
                }
                if candidate = maximal {
                    List.cons(candidate, List.cons(head, tail)).contains(maximal)
                } else {
                    tail.contains(maximal)
                    List.cons(head, tail).contains(maximal)
                    List.cons(candidate, List.cons(head, tail)).contains(maximal)
                }
                exists(result: T) {
                    result = maximal and
                    List.cons(candidate, List.cons(head, tail)).contains(result) and
                    forall(item: T) {
                        List.cons(candidate, List.cons(head, tail)).contains(item) and
                        result <= item implies item = result
                    }
                }
                p(List.cons(head, tail))
            }
        }
    }
    List.induction(p)
    p(items)
}

/// A list containing an element has a maximal element in a partial order.
theorem list_contains_has_maximal[T: PartialOrder](items: List[T], witness: T) {
    items.contains(witness) implies exists(maximal: T) {
        items.contains(maximal) and forall(item: T) {
            items.contains(item) and maximal <= item implies item = maximal
        }
    }
} by {
    if items.contains(witness) {
        match items {
            List.nil {
                not items.contains(witness)
                false
            }
            List.cons(head, tail) {
                items = List.cons(head, tail)
                list_cons_has_maximal(head, tail)
                let maximal: T satisfy {
                    List.cons(head, tail).contains(maximal) and forall(item: T) {
                        List.cons(head, tail).contains(item) and maximal <= item
                        implies item = maximal
                    }
                }
                items.contains(maximal)
                forall(item: T) {
                    if items.contains(item) and maximal <= item {
                        List.cons(head, tail).contains(item)
                        item = maximal
                    }
                }
                exists(result: T) {
                    result = maximal and items.contains(result) and forall(item: T) {
                        items.contains(item) and result <= item implies item = result
                    }
                }
            }
        }
    }
}

/// A nonempty list in a partial order has a minimal element.
theorem list_cons_has_minimal[T: PartialOrder](candidate: T, items: List[T]) {
    exists(minimal: T) {
        List.cons(candidate, items).contains(minimal) and forall(item: T) {
            List.cons(candidate, items).contains(item) and item <= minimal implies item = minimal
        }
    }
} by {
    define p(xs: List[T]) -> Bool {
        exists(minimal: T) {
            List.cons(candidate, xs).contains(minimal) and forall(item: T) {
                List.cons(candidate, xs).contains(item) and item <= minimal implies item = minimal
            }
        }
    }
    exists(minimal: T) {
        minimal = candidate and
        List.cons(candidate, List.nil[T]).contains(minimal) and forall(item: T) {
            List.cons(candidate, List.nil[T]).contains(item) and item <= minimal
            implies item = minimal
        }
    }
    p(List.nil[T])
    forall(head: T, tail: List[T]) {
        if p(tail) {
            let minimal: T satisfy {
                List.cons(candidate, tail).contains(minimal) and forall(item: T) {
                    List.cons(candidate, tail).contains(item) and item <= minimal
                    implies item = minimal
                }
            }
            if head <= minimal {
                forall(item: T) {
                    if List.cons(candidate, List.cons(head, tail)).contains(item) and
                        item <= head {
                        if item = head {
                            item = head
                        } else {
                            if candidate = item {
                                List.cons(candidate, tail).contains(item)
                            } else {
                                List.cons(head, tail).contains(item)
                                tail.contains(item)
                                List.cons(candidate, tail).contains(item)
                            }
                            lte_trans(item, head, minimal)
                            item <= minimal
                            item = minimal
                            minimal <= head
                            lte_antisymm(head, minimal)
                            head = minimal
                            false
                        }
                    }
                }
                List.cons(head, tail).contains(head)
                List.cons(candidate, List.cons(head, tail)).contains(head)
                exists(result: T) {
                    result = head and
                    List.cons(candidate, List.cons(head, tail)).contains(result) and
                    forall(item: T) {
                        List.cons(candidate, List.cons(head, tail)).contains(item) and
                        item <= result implies item = result
                    }
                }
                p(List.cons(head, tail))
            } else {
                forall(item: T) {
                    if List.cons(candidate, List.cons(head, tail)).contains(item) and
                        item <= minimal {
                        if item = head {
                            head <= minimal
                            false
                        } else {
                            if candidate = item {
                                List.cons(candidate, tail).contains(item)
                            } else {
                                List.cons(head, tail).contains(item)
                                tail.contains(item)
                                List.cons(candidate, tail).contains(item)
                            }
                            item = minimal
                        }
                    }
                }
                if candidate = minimal {
                    List.cons(candidate, List.cons(head, tail)).contains(minimal)
                } else {
                    tail.contains(minimal)
                    List.cons(head, tail).contains(minimal)
                    List.cons(candidate, List.cons(head, tail)).contains(minimal)
                }
                exists(result: T) {
                    result = minimal and
                    List.cons(candidate, List.cons(head, tail)).contains(result) and
                    forall(item: T) {
                        List.cons(candidate, List.cons(head, tail)).contains(item) and
                        item <= result implies item = result
                    }
                }
                p(List.cons(head, tail))
            }
        }
    }
    List.induction(p)
    p(items)
}

/// A list containing an element has a minimal element in a partial order.
theorem list_contains_has_minimal[T: PartialOrder](items: List[T], witness: T) {
    items.contains(witness) implies exists(minimal: T) {
        items.contains(minimal) and forall(item: T) {
            items.contains(item) and item <= minimal implies item = minimal
        }
    }
} by {
    if items.contains(witness) {
        match items {
            List.nil {
                not items.contains(witness)
                false
            }
            List.cons(head, tail) {
                items = List.cons(head, tail)
                list_cons_has_minimal(head, tail)
                let minimal: T satisfy {
                    List.cons(head, tail).contains(minimal) and forall(item: T) {
                        List.cons(head, tail).contains(item) and item <= minimal
                        implies item = minimal
                    }
                }
                items.contains(minimal)
                forall(item: T) {
                    if items.contains(item) and item <= minimal {
                        List.cons(head, tail).contains(item)
                        item = minimal
                    }
                }
                exists(result: T) {
                    result = minimal and items.contains(result) and forall(item: T) {
                        items.contains(item) and item <= result implies item = result
                    }
                }
            }
        }
    }
}

/// The head of a non-empty list is below its maximum.
theorem head_lte_list_max[T: LinearOrder](head: T, tail: List[T]) {
    head <= list_max(head, tail)
} by {
    List.cons(head, tail).contains(head)
    contains_lte_list_max(head, tail, head)
}

/// Every non-empty list has an upper bound.
theorem list_has_upper_bound[T: LinearOrder](head: T, tail: List[T]) {
    exists(ub: T) {
        is_upper_bound(List.cons(head, tail), ub)
    }
} by {
    let ub = list_max(head, tail)
}

/// True if t is a lower bound for all items in the list.
/// A lower bound t satisfies: t ≤ every element in the list.
define is_lower_bound[T: LinearOrder](list: List[T], t: T) -> Bool {
    match list {
        List.nil {
            true
        }
        List.cons(head, tail) {
            t <= head and is_lower_bound(tail, t)
        }
    }
}

/// If lb is a lower bound for a list and the list contains c, then lb ≤ c.
theorem lower_bound_contains[T: LinearOrder](list: List[T], lb: T, c: T) {
    is_lower_bound(list, lb) and list.contains(c) implies lb <= c
} by {
    define p(l: List[T]) -> Bool {
        is_lower_bound(l, lb) and l.contains(c) implies lb <= c
    }

    // Base case: empty list
    not List.nil[T].contains(c)
    p(List.nil[T])

    // Induction
    forall(head: T, tail: List[T]) {
        if p(tail) {
            if is_lower_bound(List.cons(head, tail), lb) and List.cons(head, tail).contains(c) {
                is_lower_bound(tail, lb)
                if head = c {
                    lb <= c
                } else {
                    lb <= c
                }
            }
            p(List.cons(head, tail))
        }
    }

}

/// If lb is a lower bound for (list1 + list2), then lb is a lower bound for list1.
theorem lower_bound_add_left[T: LinearOrder](list1: List[T], list2: List[T], lb: T) {
    is_lower_bound(list1 + list2, lb) implies is_lower_bound(list1, lb)
} by {
    define p(l1: List[T]) -> Bool {
        is_lower_bound(l1 + list2, lb) implies is_lower_bound(l1, lb)
    }

    // Base case: empty list
    is_lower_bound(List.nil[T], lb)
    p(List.nil[T])

    // Induction
    forall(head: T, tail: List[T]) {
        if p(tail) {
            p(List.cons(head, tail))
        }
    }

}

/// If lb is a lower bound for (list1 + list2), then lb is a lower bound for list2.
theorem lower_bound_add_right[T: LinearOrder](list1: List[T], list2: List[T], lb: T) {
    is_lower_bound(list1 + list2, lb) implies is_lower_bound(list2, lb)
} by {
    define p(l1: List[T]) -> Bool {
        is_lower_bound(l1 + list2, lb) implies is_lower_bound(list2, lb)
    }

    // Base case: empty list
    p(List.nil[T])

    // Induction
    forall(head: T, tail: List[T]) {
        if p(tail) {
            p(List.cons(head, tail))
        }
    }

}

/// If lb2 is a lower bound and lb2 ≤ lb1, then lb1 is also a lower bound.
theorem lower_bound_monotone[T: LinearOrder](list: List[T], lb1: T, lb2: T) {
    is_lower_bound(list, lb1) and lb2 <= lb1 implies is_lower_bound(list, lb2)
} by {
    define p(l: List[T]) -> Bool {
        is_lower_bound(l, lb1) and lb2 <= lb1 implies is_lower_bound(l, lb2)
    }

    // Base case: empty list
    is_lower_bound(List.nil[T], lb2)
    p(List.nil[T])

    // Induction
    forall(head: T, tail: List[T]) {
        if p(tail) {
            p(List.cons(head, tail))
        }
    }

}

/// Recursive list lower bounds are the same as membership-predicate lower bounds.
theorem lower_bound_iff_list_lower_bound[T: LinearOrder](list: List[T], lower: T) {
    is_lower_bound(list, lower) = list_lower_bound(list, lower)
} by {
    define p(l: List[T]) -> Bool {
        is_lower_bound(l, lower) = list_lower_bound(l, lower)
    }
    forall(x: T) {
        if List.nil[T].contains(x) {
            false
        }
    }
    list_lower_bound(List.nil[T], lower)
    p(List.nil[T])
    forall(head: T, tail: List[T]) {
        if p(tail) {
            if is_lower_bound(List.cons(head, tail), lower) {
                lower <= head
                is_lower_bound(tail, lower)
                list_lower_bound(tail, lower)
                forall(x: T) {
                    if List.cons(head, tail).contains(x) {
                        if head = x {
                            lower <= x
                        } else {
                            tail.contains(x)
                            lower <= x
                        }
                    }
                }
                list_lower_bound(List.cons(head, tail), lower)
            }
            if list_lower_bound(List.cons(head, tail), lower) {
                list_lower_bound(List.cons(head, tail), lower) = forall(x: T) {
                    List.cons(head, tail).contains(x) implies lower <= x
                }
                forall(x: T) {
                    if tail.contains(x) {
                        List.cons(head, tail).contains(x)
                        lower <= x
                    }
                }
                list_lower_bound(tail, lower)
                is_lower_bound(tail, lower)
                is_lower_bound(List.cons(head, tail), lower)
            }
            p(List.cons(head, tail))
        }
    }
    p(list)
}

/// Recursive list lower bounds give membership-predicate lower bounds.
theorem lower_bound_imp_list_lower_bound[T: LinearOrder](list: List[T], lower: T) {
    is_lower_bound(list, lower) implies list_lower_bound(list, lower)
} by {
    if is_lower_bound(list, lower) {
        lower_bound_iff_list_lower_bound(list, lower)
        list_lower_bound(list, lower)
    }
}

/// Membership-predicate lower bounds give recursive list lower bounds.
theorem list_lower_bound_imp_lower_bound[T: LinearOrder](list: List[T], lower: T) {
    list_lower_bound(list, lower) implies is_lower_bound(list, lower)
} by {
    if list_lower_bound(list, lower) {
        lower_bound_iff_list_lower_bound(list, lower)
        is_lower_bound(list, lower)
    }
}

/// Every point is a lower bound for the empty list.
theorem lower_bound_nil[T: LinearOrder](lower: T) {
    is_lower_bound(List.nil[T], lower)
} by {
    list_lower_bound_nil(lower)
    list_lower_bound_imp_lower_bound(List.nil[T], lower)
}

/// Lower bounds for a cons list are exactly lower bounds for the head and tail.
theorem lower_bound_cons_iff[T: LinearOrder](head: T, tail: List[T], lower: T) {
    is_lower_bound(List.cons(head, tail), lower) =
    (lower <= head and is_lower_bound(tail, lower))
} by {
    lower_bound_iff_list_lower_bound(List.cons(head, tail), lower)
    list_lower_bound_cons_iff(head, tail, lower)
    lower_bound_iff_list_lower_bound(tail, lower)
}

/// Lower bounds for a concatenation are exactly lower bounds for both lists.
theorem lower_bound_add_iff[T: LinearOrder](left: List[T], right: List[T], lower: T) {
    is_lower_bound(left + right, lower) =
    (is_lower_bound(left, lower) and is_lower_bound(right, lower))
} by {
    lower_bound_iff_list_lower_bound(left + right, lower)
    list_lower_bound_add_iff(left, right, lower)
    lower_bound_iff_list_lower_bound(left, lower)
    lower_bound_iff_list_lower_bound(right, lower)
}

/// A lower bound for a singleton list is exactly a point below the singleton element.
theorem lower_bound_singleton_iff[T: LinearOrder](item: T, lower: T) {
    is_lower_bound(List.singleton(item), lower) = (lower <= item)
} by {
    lower_bound_iff_list_lower_bound(List.singleton(item), lower)
    list_lower_bound_singleton_iff(item, lower)
}

/// Lower bounds after append are exactly lower bounds for the old list and the appended element.
theorem lower_bound_append_iff[T: LinearOrder](items: List[T], last: T, lower: T) {
    is_lower_bound(items.append(last), lower) =
    (is_lower_bound(items, lower) and lower <= last)
} by {
    lower_bound_iff_list_lower_bound(items.append(last), lower)
    list_lower_bound_append_iff(items, last, lower)
    lower_bound_iff_list_lower_bound(items, lower)
}

/// True if all elements of a list lie in the closed interval from `lower` to `upper`.
define is_interval_bound[T: LinearOrder](list: List[T], lower: T, upper: T) -> Bool {
    is_lower_bound(list, lower) and is_upper_bound(list, upper)
}

/// An interval bound applies to every element contained in the list.
theorem interval_bound_contains[T: LinearOrder](list: List[T], lower: T, upper: T, item: T) {
    is_interval_bound(list, lower, upper) and list.contains(item) implies
    closed_interval(lower, upper, item)
} by {
    if is_interval_bound(list, lower, upper) and list.contains(item) {
        is_lower_bound(list, lower)
        lower_bound_contains(list, lower, item)
        is_upper_bound(list, upper)
        upper_bound_contains(list, upper, item)
        item <= upper
        closed_interval(lower, upper, item)
    }
}

/// Lower and upper list bounds determine an interval bound.
theorem bounds_imp_interval_bound[T: LinearOrder](list: List[T], lower: T, upper: T) {
    is_lower_bound(list, lower) and is_upper_bound(list, upper) implies
    is_interval_bound(list, lower, upper)
} by {
    if is_lower_bound(list, lower) and is_upper_bound(list, upper) {
        is_interval_bound(list, lower, upper)
    }
}

/// An interval bound determines a lower bound for the list.
theorem interval_bound_imp_lower_bound[T: LinearOrder](list: List[T], lower: T, upper: T) {
    is_interval_bound(list, lower, upper) implies is_lower_bound(list, lower)
} by {
    if is_interval_bound(list, lower, upper) {
        is_lower_bound(list, lower)
    }
}

/// An interval bound determines an upper bound for the list.
theorem interval_bound_imp_upper_bound[T: LinearOrder](list: List[T], lower: T, upper: T) {
    is_interval_bound(list, lower, upper) implies is_upper_bound(list, upper)
} by {
    if is_interval_bound(list, lower, upper) {
        is_upper_bound(list, upper)
    }
}

/// Being interval-bounded is equivalent to having the corresponding lower and upper bounds.
theorem interval_bound_iff_bounds[T: LinearOrder](list: List[T], lower: T, upper: T) {
    is_interval_bound(list, lower, upper) =
    (is_lower_bound(list, lower) and is_upper_bound(list, upper))
} by {
    if is_interval_bound(list, lower, upper) {
        interval_bound_imp_lower_bound(list, lower, upper)
        interval_bound_imp_upper_bound(list, lower, upper)
        is_upper_bound(list, upper)
        is_lower_bound(list, lower) and is_upper_bound(list, upper)
    }
    if is_lower_bound(list, lower) and is_upper_bound(list, upper) {
        bounds_imp_interval_bound(list, lower, upper)
        is_interval_bound(list, lower, upper)
    }
}

/// Every ordered interval bounds the empty list.
theorem interval_bound_nil[T: LinearOrder](lower: T, upper: T) {
    is_interval_bound(List.nil[T], lower, upper)
} by {
    lower_bound_nil(lower)
    upper_bound_nil(upper)
    bounds_imp_interval_bound(List.nil[T], lower, upper)
}

/// Interval bounds for a cons list are exactly endpoint inequalities for the head and interval bounds for the tail.
theorem interval_bound_cons_iff[T: LinearOrder](head: T, tail: List[T], lower: T, upper: T) {
    is_interval_bound(List.cons(head, tail), lower, upper) =
    (closed_interval(lower, upper, head) and is_interval_bound(tail, lower, upper))
} by {
    if is_interval_bound(List.cons(head, tail), lower, upper) {
        interval_bound_imp_lower_bound(List.cons(head, tail), lower, upper)
        lower_bound_cons_iff(head, tail, lower)
        interval_bound_imp_upper_bound(List.cons(head, tail), lower, upper)
        upper_bound_cons_iff(head, tail, upper)
        bounds_imp_interval_bound(tail, lower, upper)
        is_interval_bound(tail, lower, upper)
        closed_interval(lower, upper, head) and is_interval_bound(tail, lower, upper)
    }
    if closed_interval(lower, upper, head) and is_interval_bound(tail, lower, upper) {
        head <= upper
        interval_bound_imp_lower_bound(tail, lower, upper)
        is_lower_bound(tail, lower)
        lower_bound_cons_iff(head, tail, lower)
        is_lower_bound(List.cons(head, tail), lower)
        interval_bound_imp_upper_bound(tail, lower, upper)
        upper_bound_cons_iff(head, tail, upper)
        bounds_imp_interval_bound(List.cons(head, tail), lower, upper)
        is_interval_bound(List.cons(head, tail), lower, upper)
    }
}

/// Interval bounds for a concatenation are exactly interval bounds for both lists.
theorem interval_bound_add_iff[T: LinearOrder](left: List[T], right: List[T],
        lower: T, upper: T) {
    is_interval_bound(left + right, lower, upper) =
    (is_interval_bound(left, lower, upper) and is_interval_bound(right, lower, upper))
} by {
    if is_interval_bound(left + right, lower, upper) {
        interval_bound_imp_lower_bound(left + right, lower, upper)
        lower_bound_add_iff(left, right, lower)
        interval_bound_imp_upper_bound(left + right, lower, upper)
        upper_bound_add_iff(left, right, upper)
        bounds_imp_interval_bound(left, lower, upper)
        bounds_imp_interval_bound(right, lower, upper)
        is_interval_bound(right, lower, upper)
        is_interval_bound(left, lower, upper) and is_interval_bound(right, lower, upper)
    }
    if is_interval_bound(left, lower, upper) and is_interval_bound(right, lower, upper) {
        interval_bound_imp_lower_bound(left, lower, upper)
        interval_bound_imp_lower_bound(right, lower, upper)
        is_lower_bound(right, lower)
        lower_bound_add_iff(left, right, lower)
        interval_bound_imp_upper_bound(left, lower, upper)
        interval_bound_imp_upper_bound(right, lower, upper)
        is_upper_bound(right, upper)
        upper_bound_add_iff(left, right, upper)
        bounds_imp_interval_bound(left + right, lower, upper)
        is_interval_bound(left + right, lower, upper)
    }
}

/// A singleton list is interval-bounded exactly when its element lies in the interval.
theorem interval_bound_singleton_iff[T: LinearOrder](item: T, lower: T, upper: T) {
    is_interval_bound(List.singleton(item), lower, upper) =
    closed_interval(lower, upper, item)
} by {
    interval_bound_cons_iff(item, List.nil[T], lower, upper)
    interval_bound_nil(lower, upper)
}

/// Interval bounds after append are exactly interval bounds for the old list and membership of the appended element.
theorem interval_bound_append_iff[T: LinearOrder](items: List[T], last: T,
        lower: T, upper: T) {
    is_interval_bound(items.append(last), lower, upper) =
    (is_interval_bound(items, lower, upper) and closed_interval(lower, upper, last))
} by {
    interval_bound_add_iff(items, List.singleton(last), lower, upper)
    interval_bound_singleton_iff(last, lower, upper)
}

/// A list lower bound is a lower bound for the list membership predicate.
theorem lower_bound_imp_contains_lower_bound[T: LinearOrder](list: List[T], lower: T) {
    is_lower_bound(list, lower) implies
    lib(order).is_lower_bound(list.contains, lower)
} by {
    if is_lower_bound(list, lower) {
        forall(item: T) {
            if list.contains(item) {
                lower_bound_contains(list, lower, item)
                lower <= item
            }
        }
        lib(order).is_lower_bound(list.contains, lower)
    }
}

/// A lower bound for the list membership predicate is a list lower bound.
theorem contains_lower_bound_imp_lower_bound[T: LinearOrder](list: List[T], lower: T) {
    lib(order).is_lower_bound(list.contains, lower) implies
    is_lower_bound(list, lower)
} by {
    if lib(order).is_lower_bound(list.contains, lower) {
        forall(item: T) {
            if list.contains(item) {
                lib(order).lower_bound_step(list.contains, lower, item)
                lower <= item
            }
        }
        list_lower_bound_imp_lower_bound(list, lower)
        is_lower_bound(list, lower)
    }
}

/// List lower bounds are exactly lower bounds for the list membership predicate.
theorem lower_bound_iff_contains_lower_bound[T: LinearOrder](list: List[T], lower: T) {
    is_lower_bound(list, lower) =
    lib(order).is_lower_bound(list.contains, lower)
} by {
    if is_lower_bound(list, lower) {
        lower_bound_imp_contains_lower_bound(list, lower)
        lib(order).is_lower_bound(list.contains, lower)
    }
    if lib(order).is_lower_bound(list.contains, lower) {
        contains_lower_bound_imp_lower_bound(list, lower)
        is_lower_bound(list, lower)
    }
}

/// A list lower bound gives boundedness below for the list membership predicate.
theorem lower_bound_imp_contains_bounded_below[T: LinearOrder](list: List[T], lower: T) {
    is_lower_bound(list, lower) implies
    lib(order).is_bounded_below(list.contains)
} by {
    if is_lower_bound(list, lower) {
        lower_bound_imp_contains_lower_bound(list, lower)
        lib(order).lower_bound_imp_bounded_below(list.contains, lower)
        lib(order).is_bounded_below(list.contains)
    }
}

/// A list upper bound is an upper bound for the list membership predicate.
theorem upper_bound_imp_contains_upper_bound[T: LinearOrder](list: List[T], upper: T) {
    is_upper_bound(list, upper) implies
    lib(order).is_upper_bound(list.contains, upper)
} by {
    if is_upper_bound(list, upper) {
        forall(item: T) {
            if list.contains(item) {
                upper_bound_contains(list, upper, item)
                item <= upper
            }
        }
        lib(order).is_upper_bound(list.contains, upper)
    }
}

/// An upper bound for the list membership predicate is a list upper bound.
theorem contains_upper_bound_imp_upper_bound[T: LinearOrder](list: List[T], upper: T) {
    lib(order).is_upper_bound(list.contains, upper) implies
    is_upper_bound(list, upper)
} by {
    if lib(order).is_upper_bound(list.contains, upper) {
        forall(item: T) {
            if list.contains(item) {
                lib(order).upper_bound_step(list.contains, upper, item)
                item <= upper
            }
        }
        list_upper_bound_imp_upper_bound(list, upper)
        is_upper_bound(list, upper)
    }
}

/// List upper bounds are exactly upper bounds for the list membership predicate.
theorem upper_bound_iff_contains_upper_bound[T: LinearOrder](list: List[T], upper: T) {
    is_upper_bound(list, upper) =
    lib(order).is_upper_bound(list.contains, upper)
} by {
    if is_upper_bound(list, upper) {
        upper_bound_imp_contains_upper_bound(list, upper)
        lib(order).is_upper_bound(list.contains, upper)
    }
    if lib(order).is_upper_bound(list.contains, upper) {
        contains_upper_bound_imp_upper_bound(list, upper)
        is_upper_bound(list, upper)
    }
}

/// A list upper bound gives boundedness above for the list membership predicate.
theorem upper_bound_imp_contains_bounded_above[T: LinearOrder](list: List[T], upper: T) {
    is_upper_bound(list, upper) implies
    lib(order).is_bounded_above(list.contains)
} by {
    if is_upper_bound(list, upper) {
        upper_bound_imp_contains_upper_bound(list, upper)
        lib(order).upper_bound_imp_bounded_above(list.contains, upper)
        lib(order).is_bounded_above(list.contains)
    }
}

/// An interval bound for a list is an interval bound for its membership predicate.
theorem interval_bound_imp_contains_bounded_by_interval[T: LinearOrder](list: List[T],
        lower: T, upper: T) {
    is_interval_bound(list, lower, upper) implies
    lib(order).is_bounded_by_interval(list.contains, lower, upper)
} by {
    if is_interval_bound(list, lower, upper) {
        forall(item: T) {
            if list.contains(item) {
                interval_bound_contains(list, lower, upper, item)
                closed_interval(lower, upper, item)
            }
        }
        lib(order).is_bounded_by_interval(list.contains, lower, upper)
    }
}

/// An interval bound for the list membership predicate is an interval bound for the list.
theorem contains_bounded_by_interval_imp_interval_bound[T: LinearOrder](list: List[T],
        lower: T, upper: T) {
    lib(order).is_bounded_by_interval(list.contains, lower, upper) implies
    is_interval_bound(list, lower, upper)
} by {
    if lib(order).is_bounded_by_interval(list.contains, lower, upper) {
        lib(order).bounded_by_interval_imp_lower_bound(list.contains, lower, upper)
        contains_lower_bound_imp_lower_bound(list, lower)
        lib(order).bounded_by_interval_imp_upper_bound(list.contains, lower, upper)
        contains_upper_bound_imp_upper_bound(list, upper)
        bounds_imp_interval_bound(list, lower, upper)
        is_interval_bound(list, lower, upper)
    }
}

/// List interval bounds are exactly interval bounds for the membership predicate.
theorem interval_bound_iff_contains_bounded_by_interval[T: LinearOrder](list: List[T],
        lower: T, upper: T) {
    is_interval_bound(list, lower, upper) =
    lib(order).is_bounded_by_interval(list.contains, lower, upper)
} by {
    if is_interval_bound(list, lower, upper) {
        interval_bound_imp_contains_bounded_by_interval(list, lower, upper)
        lib(order).is_bounded_by_interval(list.contains, lower, upper)
    }
    if lib(order).is_bounded_by_interval(list.contains, lower, upper) {
        contains_bounded_by_interval_imp_interval_bound(list, lower, upper)
        is_interval_bound(list, lower, upper)
    }
}

/// An interval bound for a list gives boundedness for its membership predicate.
theorem interval_bound_imp_contains_bounded[T: LinearOrder](list: List[T],
        lower: T, upper: T) {
    is_interval_bound(list, lower, upper) implies
    lib(order).is_bounded(list.contains)
} by {
    if is_interval_bound(list, lower, upper) {
        interval_bound_imp_contains_bounded_by_interval(list, lower, upper)
        lib(order).bounded_by_interval_imp_bounded(list.contains, lower, upper)
        lib(order).is_bounded(list.contains)
    }
}

/// The minimum element of a non-empty list.
define list_min[T: LinearOrder](head: T, tail: List[T]) -> T {
    match tail {
        List.nil {
            head
        }
        List.cons(next, rest) {
            head.min(list_min(next, rest))
        }
    }
}

/// The minimum element of a list is a lower bound for that list.
theorem list_min_is_lower_bound[T: LinearOrder](head: T, tail: List[T]) {
    is_lower_bound(List.cons(head, tail), list_min(head, tail))
} by {
    define p(t: List[T]) -> Bool {
        forall(h: T) {
            is_lower_bound(List.cons(h, t), list_min(h, t))
        }
    }

    // Base case: tail is nil (singleton list)
    forall(h: T) {
        h <= h
        is_lower_bound(List.cons(h, List.nil[T]), list_min(h, List.nil[T]))
    }
    p(List.nil[T])

    // Inductive case
    forall(next: T, rest: List[T]) {
        if p(rest) {
            forall(h: T) {
                let m = list_min(h, List.cons(next, rest))
                is_lower_bound(List.cons(next, rest), list_min(next, rest))
                h.min(list_min(next, rest)) = list_min(h, List.cons(next, rest))
                h.min(list_min(next, rest)) <= list_min(next, rest)
                h.min(list_min(next, rest)) <= h
                list_min(h, List.cons(next, rest)) <= h
                is_lower_bound(List.cons(next, rest), m)
                is_lower_bound(List.cons(h, List.cons(next, rest)), list_min(h, List.cons(next, rest)))
            }
            // We've shown forall(h: T) { is_lower_bound(List.cons(h, List.cons(next, rest)), list_min(h, List.cons(next, rest))) }
            // which is exactly p(List.cons(next, rest))
            p(List.cons(next, rest))
        }
    }

    // Apply to our specific head and tail
}

/// Any lower bound for every element of a non-empty list is below its minimum.
theorem lte_list_min_of_lower_bound[T: LinearOrder](lower: T, head: T, tail: List[T]) {
    is_lower_bound(List.cons(head, tail), lower) implies
    lower <= list_min(head, tail)
} by {
    define p(t: List[T]) -> Bool {
        forall(h: T, bound: T) {
            is_lower_bound(List.cons(h, t), bound) implies
            bound <= list_min(h, t)
        }
    }
    forall(h: T, bound: T) {
        if is_lower_bound(List.cons(h, List.nil[T]), bound) {
            lower_bound_contains(List.cons(h, List.nil[T]), bound, h)
            bound <= list_min(h, List.nil[T])
        }
    }
    p(List.nil[T])
    forall(next: T, rest: List[T]) {
        if p(rest) {
            forall(h: T, bound: T) {
                if is_lower_bound(List.cons(h, List.cons(next, rest)), bound) {
                    lower_bound_add_right(List.singleton(h), List.cons(next, rest), bound)
                    is_lower_bound(List.cons(next, rest), bound)
                    lower_bound_contains(List.cons(h, List.cons(next, rest)), bound, h)
                    bound <= h
                    lte_min_of_bounds(bound, h, list_min(next, rest))
                    bound <= h.min(list_min(next, rest))
                    bound <= list_min(h, List.cons(next, rest))
                }
            }
            p(List.cons(next, rest))
        }
    }
    p(tail)
}

/// The minimum of a non-empty list is the greatest lower bound of its elements.
theorem lte_list_min_iff_lower_bound[T: LinearOrder](lower: T, head: T, tail: List[T]) {
    lower <= list_min(head, tail) = is_lower_bound(List.cons(head, tail), lower)
} by {
    if lower <= list_min(head, tail) {
        list_min_is_lower_bound(head, tail)
        lower_bound_monotone(List.cons(head, tail), list_min(head, tail), lower)
        is_lower_bound(List.cons(head, tail), lower)
    }
    if is_lower_bound(List.cons(head, tail), lower) {
        lte_list_min_of_lower_bound(lower, head, tail)
        lower <= list_min(head, tail)
    }
}

/// A point is below the list minimum exactly when it is a lower bound.
theorem le_list_min_iff[T: LinearOrder](lower: T, head: T, tail: List[T]) {
    lower <= list_min(head, tail) = is_lower_bound(List.cons(head, tail), lower)
} by {
    lte_list_min_iff_lower_bound(lower, head, tail)
}

/// The minimum of a non-empty list is a membership-predicate lower bound.
theorem list_min_is_list_lower_bound[T: LinearOrder](head: T, tail: List[T]) {
    list_lower_bound(List.cons(head, tail), list_min(head, tail))
} by {
    list_min_is_lower_bound(head, tail)
    lower_bound_imp_list_lower_bound(List.cons(head, tail), list_min(head, tail))
}

/// A point is below the list minimum exactly when it is a membership-predicate lower bound.
theorem le_list_min_iff_list_lower_bound[T: LinearOrder](lower: T, head: T, tail: List[T]) {
    lower <= list_min(head, tail) = list_lower_bound(List.cons(head, tail), lower)
} by {
    lte_list_min_iff_lower_bound(lower, head, tail)
    lower_bound_iff_list_lower_bound(List.cons(head, tail), lower)
}

/// The minimum of a non-empty list belongs to the list.
theorem list_min_contains[T: LinearOrder](head: T, tail: List[T]) {
    List.cons(head, tail).contains(list_min(head, tail))
} by {
    define p(items: List[T]) -> Bool {
        forall(first: T) {
            List.cons(first, items).contains(list_min(first, items))
        }
    }
    forall(first: T) {
        list_min(first, List.nil[T]) = first
        List.cons(first, List.nil[T]).contains(first)
    }
    p(List.nil[T])
    forall(next: T, rest: List[T]) {
        if p(rest) {
            forall(first: T) {
                list_min(first, List.cons(next, rest)) =
                    first.min(list_min(next, rest))
                min_is_one(first, list_min(next, rest))
                if first.min(list_min(next, rest)) = first {
                    List.cons(first, List.cons(next, rest)).contains(
                        list_min(first, List.cons(next, rest)))
                } else {
                    first.min(list_min(next, rest)) = list_min(next, rest)
                    list_min(first, List.cons(next, rest)) = list_min(next, rest)
                    List.cons(next, rest).contains(list_min(next, rest))
                    List.cons(first, List.cons(next, rest)).contains(
                        list_min(first, List.cons(next, rest)))
                }
            }
            p(List.cons(next, rest))
        }
    }
    List.induction(p)
    p(tail)
}

/// The minimum of a non-empty list is below every element of the list.
theorem list_min_lte_contains[T: LinearOrder](head: T, tail: List[T], item: T) {
    List.cons(head, tail).contains(item) implies list_min(head, tail) <= item
} by {
    if List.cons(head, tail).contains(item) {
        list_min_is_lower_bound(head, tail)
        lower_bound_contains(List.cons(head, tail), list_min(head, tail), item)
        list_min(head, tail) <= item
    }
}

/// A list containing an element has a least element.
theorem list_contains_has_least[T: LinearOrder](items: List[T], witness: T) {
    items.contains(witness) implies exists(least: T) {
        items.contains(least) and forall(item: T) {
            items.contains(item) implies least <= item
        }
    }
} by {
    define p(xs: List[T]) -> Bool {
        xs.contains(witness) implies exists(least: T) {
            xs.contains(least) and forall(item: T) {
                xs.contains(item) implies least <= item
            }
        }
    }
    not List.nil[T].contains(witness)
    p(List.nil[T])
    forall(head: T, tail: List[T]) {
        if p(tail) {
            if List.cons(head, tail).contains(witness) {
                list_min_contains(head, tail)
                forall(item: T) {
                    if List.cons(head, tail).contains(item) {
                        list_min_lte_contains(head, tail, item)
                    }
                }
                exists(least: T) {
                    least = list_min(head, tail) and
                    List.cons(head, tail).contains(least) and forall(item: T) {
                        List.cons(head, tail).contains(item) implies least <= item
                    }
                }
            }
            p(List.cons(head, tail))
        }
    }
    List.induction(p)
    p(items)
}

/// The minimum of a non-empty list is below its head.
theorem list_min_lte_head[T: LinearOrder](head: T, tail: List[T]) {
    list_min(head, tail) <= head
} by {
    List.cons(head, tail).contains(head)
    list_min_lte_contains(head, tail, head)
}

/// The minimum over a concatenation with a non-empty right list is the minimum of the two list minima.
theorem list_min_add_cons[T: LinearOrder](head: T, tail: List[T], next: T, rest: List[T]) {
    list_min(head, tail + List.cons(next, rest)) =
    list_min(head, tail).min(list_min(next, rest))
} by {
    define p(t: List[T]) -> Bool {
        forall(h: T, n: T, r: List[T]) {
            list_min(h, t + List.cons(n, r)) =
            list_min(h, t).min(list_min(n, r))
        }
    }
    forall(h: T, n: T, r: List[T]) {
        list_min(h, List.nil[T] + List.cons(n, r)) =
        list_min(h, List.nil[T]).min(list_min(n, r))
    }
    p(List.nil[T])
    forall(next_tail: T, rest_tail: List[T]) {
        if p(rest_tail) {
            forall(h: T, n: T, r: List[T]) {
                let right_min = list_min(n, r)
                let tail_min = list_min(next_tail, rest_tail)
                list_min(h, List.cons(next_tail, rest_tail) + List.cons(n, r)) =
                h.min(list_min(next_tail, rest_tail + List.cons(n, r)))
                list_min(next_tail, rest_tail + List.cons(n, r)) =
                list_min(next_tail, rest_tail).min(list_min(n, r))
                min_assoc_rev(h, tail_min, right_min)
                list_min(h, List.cons(next_tail, rest_tail) + List.cons(n, r)) =
                list_min(h, List.cons(next_tail, rest_tail)).min(list_min(n, r))
            }
            p(List.cons(next_tail, rest_tail))
        }
    }
    p(tail)
}

/// The maximum over a concatenation with a non-empty right list is the maximum of the two list maxima.
theorem list_max_add_cons[T: LinearOrder](head: T, tail: List[T], next: T, rest: List[T]) {
    list_max(head, tail + List.cons(next, rest)) =
    list_max(head, tail).max(list_max(next, rest))
} by {
    define p(t: List[T]) -> Bool {
        forall(h: T, n: T, r: List[T]) {
            list_max(h, t + List.cons(n, r)) =
            list_max(h, t).max(list_max(n, r))
        }
    }
    forall(h: T, n: T, r: List[T]) {
        list_max(h, List.nil[T] + List.cons(n, r)) =
        list_max(h, List.nil[T]).max(list_max(n, r))
    }
    p(List.nil[T])
    forall(next_tail: T, rest_tail: List[T]) {
        if p(rest_tail) {
            forall(h: T, n: T, r: List[T]) {
                let right_max = list_max(n, r)
                let tail_max = list_max(next_tail, rest_tail)
                list_max(h, List.cons(next_tail, rest_tail) + List.cons(n, r)) =
                h.max(list_max(next_tail, rest_tail + List.cons(n, r)))
                list_max(next_tail, rest_tail + List.cons(n, r)) =
                list_max(next_tail, rest_tail).max(list_max(n, r))
                max_assoc_rev(h, tail_max, right_max)
                list_max(h, List.cons(next_tail, rest_tail) + List.cons(n, r)) =
                list_max(h, List.cons(next_tail, rest_tail)).max(list_max(n, r))
            }
            p(List.cons(next_tail, rest_tail))
        }
    }
    p(tail)
}

/// The minimum over a concatenation is below the minimum of the left non-empty list.
theorem list_min_add_cons_lte_left[T: LinearOrder](head: T, tail: List[T], next: T, rest: List[T]) {
    list_min(head, tail + List.cons(next, rest)) <= list_min(head, tail)
} by {
    list_min_add_cons(head, tail, next, rest)
    min_lte_left(list_min(head, tail), list_min(next, rest))
}

/// The minimum over a concatenation is below the minimum of the right non-empty list.
theorem list_min_add_cons_lte_right[T: LinearOrder](head: T, tail: List[T], next: T, rest: List[T]) {
    list_min(head, tail + List.cons(next, rest)) <= list_min(next, rest)
} by {
    list_min_add_cons(head, tail, next, rest)
    min_lte_right(list_min(head, tail), list_min(next, rest))
}

/// The maximum of the left non-empty list is below the maximum over a concatenation.
theorem list_max_lte_add_cons_left[T: LinearOrder](head: T, tail: List[T], next: T, rest: List[T]) {
    list_max(head, tail) <= list_max(head, tail + List.cons(next, rest))
} by {
    list_max_add_cons(head, tail, next, rest)
    lte_max_left(list_max(head, tail), list_max(next, rest))
}

/// The maximum of the right non-empty list is below the maximum over a concatenation.
theorem list_max_lte_add_cons_right[T: LinearOrder](head: T, tail: List[T], next: T, rest: List[T]) {
    list_max(next, rest) <= list_max(head, tail + List.cons(next, rest))
} by {
    list_max_add_cons(head, tail, next, rest)
    lte_max_right(list_max(head, tail), list_max(next, rest))
}

/// Appending one element takes the minimum of the old list minimum and that element.
theorem list_min_append[T: LinearOrder](head: T, tail: List[T], last: T) {
    list_min(head, tail.append(last)) = list_min(head, tail).min(last)
} by {
    List.singleton(last) = List.cons(last, List.nil[T])
    list_min_add_cons(head, tail, last, List.nil[T])
}

/// Appending one element takes the maximum of the old list maximum and that element.
theorem list_max_append[T: LinearOrder](head: T, tail: List[T], last: T) {
    list_max(head, tail.append(last)) = list_max(head, tail).max(last)
} by {
    List.singleton(last) = List.cons(last, List.nil[T])
    list_max_add_cons(head, tail, last, List.nil[T])
}

/// The minimum after appending an element is below the previous list minimum.
theorem list_min_append_lte_list_min[T: LinearOrder](head: T, tail: List[T], last: T) {
    list_min(head, tail.append(last)) <= list_min(head, tail)
} by {
    list_min_append(head, tail, last)
    min_lte_left(list_min(head, tail), last)
}

/// The minimum after appending an element is below that element.
theorem list_min_append_lte_last[T: LinearOrder](head: T, tail: List[T], last: T) {
    list_min(head, tail.append(last)) <= last
} by {
    list_min_append(head, tail, last)
    min_lte_right(list_min(head, tail), last)
}

/// The previous list maximum is below the maximum after appending an element.
theorem list_max_lte_append_max[T: LinearOrder](head: T, tail: List[T], last: T) {
    list_max(head, tail) <= list_max(head, tail.append(last))
} by {
    list_max_append(head, tail, last)
    lte_max_left(list_max(head, tail), last)
}

/// The appended element is below the maximum after appending it.
theorem last_lte_list_max_append[T: LinearOrder](head: T, tail: List[T], last: T) {
    last <= list_max(head, tail.append(last))
} by {
    list_max_append(head, tail, last)
    lte_max_right(list_max(head, tail), last)
}

/// Every non-empty list has a lower bound.
theorem list_has_lower_bound[T: LinearOrder](head: T, tail: List[T]) {
    exists(lb: T) {
        is_lower_bound(List.cons(head, tail), lb)
    }
} by {
    let lb = list_min(head, tail)
}

/// The minimum and maximum of a non-empty list bound every element of the list by a closed interval.
theorem list_min_max_interval_bound[T: LinearOrder](head: T, tail: List[T]) {
    is_interval_bound(List.cons(head, tail), list_min(head, tail), list_max(head, tail))
} by {
    list_min_is_lower_bound(head, tail)
    list_max_is_upper_bound(head, tail)
    bounds_imp_interval_bound(List.cons(head, tail), list_min(head, tail), list_max(head, tail))
}

/// The list membership predicate is interval-bounded by the minimum and maximum.
theorem list_min_max_contains_bounded_by_interval[T: LinearOrder](head: T, tail: List[T]) {
    lib(order).is_bounded_by_interval(List.cons(head, tail).contains,
        list_min(head, tail), list_max(head, tail))
} by {
    list_min_max_interval_bound(head, tail)
    interval_bound_imp_contains_bounded_by_interval(List.cons(head, tail),
        list_min(head, tail), list_max(head, tail))
}

/// The list membership predicate is bounded.
theorem list_min_max_contains_bounded[T: LinearOrder](head: T, tail: List[T]) {
    lib(order).is_bounded(List.cons(head, tail).contains)
} by {
    list_min_max_contains_bounded_by_interval(head, tail)
    lib(order).bounded_by_interval_imp_bounded(List.cons(head, tail).contains,
        list_min(head, tail), list_max(head, tail))
}

/// The list membership predicate is bounded below by the minimum.
theorem list_min_contains_lower_bound[T: LinearOrder](head: T, tail: List[T]) {
    lib(order).is_lower_bound(List.cons(head, tail).contains, list_min(head, tail))
} by {
    list_min_is_lower_bound(head, tail)
    lower_bound_imp_contains_lower_bound(List.cons(head, tail), list_min(head, tail))
}

/// The list membership predicate is bounded below.
theorem list_min_contains_bounded_below[T: LinearOrder](head: T, tail: List[T]) {
    lib(order).is_bounded_below(List.cons(head, tail).contains)
} by {
    list_min_contains_lower_bound(head, tail)
    lib(order).lower_bound_imp_bounded_below(List.cons(head, tail).contains,
        list_min(head, tail))
}

/// The list membership predicate is bounded above by the maximum.
theorem list_max_contains_upper_bound[T: LinearOrder](head: T, tail: List[T]) {
    lib(order).is_upper_bound(List.cons(head, tail).contains, list_max(head, tail))
} by {
    list_max_is_upper_bound(head, tail)
    upper_bound_imp_contains_upper_bound(List.cons(head, tail), list_max(head, tail))
}

/// The list membership predicate is bounded above.
theorem list_max_contains_bounded_above[T: LinearOrder](head: T, tail: List[T]) {
    lib(order).is_bounded_above(List.cons(head, tail).contains)
} by {
    list_max_contains_upper_bound(head, tail)
    lib(order).upper_bound_imp_bounded_above(List.cons(head, tail).contains,
        list_max(head, tail))
}

/// Every element of a non-empty list lies between the list minimum and maximum.
theorem list_contains_closed_interval_min_max[T: LinearOrder](head: T, tail: List[T], item: T) {
    List.cons(head, tail).contains(item) implies
    closed_interval(list_min(head, tail), list_max(head, tail), item)
} by {
    if List.cons(head, tail).contains(item) {
        list_min_max_interval_bound(head, tail)
        interval_bound_contains(List.cons(head, tail), list_min(head, tail), list_max(head, tail), item)
        closed_interval(list_min(head, tail), list_max(head, tail), item)
    }
}

/// A non-empty list is interval-bounded by its minimum and maximum.
theorem list_has_interval_bound[T: LinearOrder](head: T, tail: List[T]) {
    exists(lower: T, upper: T) {
        is_interval_bound(List.cons(head, tail), lower, upper)
    }
} by {
    list_min_max_interval_bound(head, tail)
}
