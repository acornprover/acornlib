from nat import Nat, alt_suc_ne_zero, lte_trans, not_lt_zero, only_zero_lte_zero,
    zero_or_suc
from list.list_base import List, map, unique_is_smallest_containing_list
from algebra.add_comm_monoid import AddCommMonoid
from algebra.add_semigroup import add_fn
from semiring import Semiring
from algebra.semigroup import mul_fn
from data.basic.functions import compose, is_injective_fn, injective_fn_eq

numerals Nat

/// Computes the sum of all elements in a list (requires elements to form an additive commutative monoid).
define sum[A: AddCommMonoid](items: List[A]) -> A {
    match items {
        List.nil {
            A.0
        }
        List.cons(head, tail) {
            head + sum(tail)
        }
    }
}

/// Computes the partial sum of a series up to index n.
/// Returns the sum of f(0) + f(1) + ... + f(n-1).
define partial[A: AddCommMonoid](f: Nat -> A, n: Nat) -> A {
    sum(map(n.range, f))
}

/// The partial sum of a single element equals that element.
theorem partial_one[A: AddCommMonoid](f: Nat -> A) {
    partial(f, 1) = f(0)
} by {
    sum[A](map[Nat, A](1.range, f)) = partial[A](f, 1)
    List.nil[Nat] + List.singleton(0) = List.nil[Nat].append(0)
    List.cons(f(0), List.nil[A]) = List.singleton(f(0))
    f(0) + A.0 = f(0)
    List.nil[Nat] + List.singleton(0) = List.singleton(0)
    0.range.append(0) = 0.suc.range
    f(0) + sum[A](List.nil[A]) = sum[A](List.cons(f(0), List.nil[A]))
    List.cons(0, List.nil[Nat]) = List.singleton(0)
    map[Nat, A](List.nil[Nat], f) = List.nil[A]
    List.cons(f(0), map[Nat, A](List.nil[Nat], f)) = map[Nat, A](List.cons(0, List.nil[Nat]), f)
    map(List.singleton(0), f) = List.singleton(f(0))
}

/// Summing two mapped lists equals mapping with the pointwise sum of functions.
theorem map_sum_add[T, A: AddCommMonoid](list: List[T], f: T -> A, g: T -> A) {
    sum(map(list, f)) + sum(map(list, g)) = sum(map(list, add_fn(f, g)))
} by {
    define p(l: List[T]) -> Bool {
        sum(map(l, f)) + sum(map(l, g)) = sum(map(l, add_fn(f, g)))
    }

    // Base case: empty list
    map[T, A](List.nil[T], f) = List.nil[A]
    map[T, A](List.nil[T], g) = List.nil[A]
    map[T, A](List.nil[T], add_fn(f, g)) = List.nil[A]
    sum[A](List.nil[A]) = A.0
    A.0 + sum[A](List.nil[A]) = sum[A](List.nil[A])
    p(List.nil)

    // Inductive step
    forall(head: T, tail: List[T]) {
        if p(tail) {
            // Induction hypothesis: sum(map(tail, f)) + sum(map(tail, g)) = sum(map(tail, add_fn(f, g)))

            // LHS: sum(map(List.cons(head, tail), f)) + sum(map(List.cons(head, tail), g))

            // RHS: sum(map(List.cons(head, tail), add_fn(f, g)))

            // Now we need to show:
            // (f(head) + sum(map(tail, f))) + (g(head) + sum(map(tail, g))) = (f(head) + g(head)) + sum(map(tail, add_fn(f, g)))

            // Using the induction hypothesis:

            // Using associativity and commutativity:
            f(head) + (g(head) + (sum(map(tail, f)) + sum(map(tail, g)))) = (f(head) + g(head)) + (sum(map(tail, f)) + sum(map(tail, g)))
            (f(head) + g(head)) + (sum(map(tail, f)) + sum(map(tail, g))) = (f(head) + g(head)) + sum(map(tail, add_fn(f, g)))

            // Explicate map expansions
            map(List.cons(head, tail), f) = List.cons(f(head), map(tail, f))
            map(List.cons(head, tail), g) = List.cons(g(head), map(tail, g))
            map(List.cons(head, tail), add_fn(f, g)) = List.cons(add_fn(f, g, head), map(tail, add_fn(f, g)))

            // Explicate add_fn definition
            f(head) + g(head) = add_fn(f, g, head)

            // Explicate sum expansions

            // Explicate commutativity

            // Explicate associativity
            f(head) + sum(map(tail, f)) + (g(head) + sum(map(tail, g))) = f(head) + (sum(map(tail, f)) + (g(head) + sum(map(tail, g))))
            g(head) + sum(map(tail, f)) + sum(map(tail, g)) = g(head) + (sum(map(tail, f)) + sum(map(tail, g)))
            sum(map(tail, f)) + (g(head) + sum(map(tail, g))) = sum(map(tail, f)) + g(head) + sum(map(tail, g))

            p(List.cons(head, tail))
        }
    }
}

/// Mapped sums agree when the mapped functions agree on every list element.
theorem sum_map_of_pointwise[T, A: AddCommMonoid](items: List[T], f: T -> A, g: T -> A) {
    forall(x: T) { items.contains(x) implies f(x) = g(x) } implies
        sum(map(items, f)) = sum(map(items, g))
} by {
    define p(xs: List[T]) -> Bool {
        forall(x: T) { xs.contains(x) implies f(x) = g(x) } implies
            sum(map(xs, f)) = sum(map(xs, g))
    }

    if forall(x: T) { List.nil[T].contains(x) implies f(x) = g(x) } {
        sum(map(List.nil[T], f)) = sum(map(List.nil[T], g))
    }
    p(List.nil[T])

    forall(head: T, tail: List[T]) {
        if p(tail) {
            if forall(x: T) { List.cons(head, tail).contains(x) implies f(x) = g(x) } {
                forall(x: T) {
                    if tail.contains(x) {
                        List.cons(head, tail).contains(x)
                        f(x) = g(x)
                    }
                }
                if forall(x: T) { tail.contains(x) implies f(x) = g(x) } {
                    p(tail) = (forall(x: T) { tail.contains(x) implies f(x) = g(x) } implies
                        sum(map(tail, f)) = sum(map(tail, g)))
                    sum(map(tail, f)) = sum(map(tail, g))
                }
                f(head) = g(head)
                g(head) + sum(map(tail, f)) = g(head) + sum(map(tail, g))
                sum(map(List.cons(head, tail), f)) = sum(map(List.cons(head, tail), g))
            }
            p(List.cons(head, tail))
        }
    }
    List.induction(function(xs: List[T]) { p(xs) })
    p(items)
}

/// Adding two partial sums equals the partial sum of pointwise sums.
theorem partial_add[A: AddCommMonoid](f: Nat -> A, g: Nat -> A, n: Nat) {
    partial(f, n) + partial(g, n) = partial(add_fn(f, g), n)
} by {
    // Expand the definitions of partial

    // Use map_sum_add theorem
}

/// Helper function for scalar multiplication.
define scalar_mul[S: Semiring](c: S, x: S) -> S {
    c * x
}

/// Multiplying a constant by a sum equals the sum of the products.
theorem sum_scalar_mul[S: Semiring](c: S, list: List[S]) {
    c * sum(list) = sum(map(list, scalar_mul(c)))
} by {
    define p(xs: List[S]) -> Bool {
        c * sum(xs) = sum(map(xs, scalar_mul(c)))
    }

    // Base case: c * sum(nil) = sum(map(nil, scalar_mul(c)))
    p(List.nil)

    // Inductive step
    forall(head: S, tail: List[S]) {
        if p(tail) {
            // Induction hypothesis: c * sum(tail) = sum(map(tail, scalar_mul(c)))

            // Left side: c * sum(cons(head, tail))

            // Apply induction hypothesis

            // Right side: sum(map(cons(head, tail), scalar_mul(c)))

            c * head + c * sum(tail) = c * (head + sum(tail))
            sum[S](List.cons(head, tail)) = head + sum[S](tail)
            c * head = scalar_mul(c, head)
            List.cons(scalar_mul(c, head), map[S, S](tail, scalar_mul(c))) = map[S, S](List.cons(head, tail), scalar_mul(c))
            sum[S](List.cons(scalar_mul(c, head), map[S, S](tail, scalar_mul(c)))) = scalar_mul(c, head) + sum[S](map[S, S](tail, scalar_mul(c)))

            c * sum(List.cons(head, tail)) = sum(map(List.cons(head, tail), scalar_mul(c)))
            p(List.cons(head, tail))
        }
    }

    forall(l: List[S]) { p(l) }
}

theorem sum_add[A: AddCommMonoid](left: List[A], right: List[A]) {
    sum(left + right) = sum(left) + sum(right)
} by {
    define p(x: List[A]) -> Bool {
        sum(x + right) = sum(x) + sum(right)
    }

    // Base case: sum(nil + right) = sum(nil) + sum(right)
    p(List.nil)

    // Inductive step
    forall(head: A, tail: List[A]) {
        if p(tail) {
            // Induction hypothesis: sum(tail + right) = sum(tail) + sum(right)

            // Left side: sum(List.cons(head, tail) + right)

            // Use induction hypothesis

            // Right side: sum(List.cons(head, tail)) + sum(right)

            // Use associativity

            // Therefore
            p(List.cons(head, tail))
        }
    }
}

theorem map_add[T, U](left: List[T], right: List[T], f: T -> U) {
    map(left + right, f) = map(left, f) + map(right, f)
} by {
    define p(x: List[T]) -> Bool {
        map(x + right, f) = map(x, f) + map(right, f)
    }

    // Base case: map(nil + right, f) = map(nil, f) + map(right, f)
    p(List.nil)

    // Inductive step

    forall(head: T, tail: List[T]) {
        if p(tail) {
            // Induction hypothesis: map(tail + right, f) = map(tail, f) + map(right, f)
            p(tail) implies map[T, U](tail, f) + map[T, U](right, f) = map[T, U](tail + right, f)
            List.cons(f(head), map[T, U](tail, f)) + map[T, U](right, f) = List.cons(f(head), map[T, U](tail, f) + map[T, U](right, f))
            List.cons(f(head), map[T, U](tail + right, f)) = map[T, U](List.cons(head, tail + right), f)
            p(List.cons(head, tail))
        }
    }
}

theorem map_append[T, U](initial: List[T], last: T, f: T -> U) {
    map(initial.append(last), f) = map(initial, f).append(f(last))
} by {
    List.cons(f(last), List.nil[U]) = List.singleton(f(last))
    List.cons(last, List.nil[T]) = List.singleton(last)
    List.cons(f(last), map[T, U](List.nil[T], f)) = map[T, U](List.cons(last, List.nil[T]), f)
}

theorem sum_append[A: AddCommMonoid](initial: List[A], last: A) {
    sum(initial.append(last)) = sum(initial) + last
}

theorem add_assoc[T](a: List[T], b: List[T], c: List[T]) {
    (a + b) + c = a + (b + c)
} by {
    define p(x: List[T]) -> Bool {
        (x + b) + c = x + (b + c)
    }

    // Base case: (nil + b) + c = nil + (b + c)
    p(List.nil)

    // Inductive step
    forall(head: T, tail: List[T]) {
        if p(tail) {
            // Induction hypothesis: (tail + b) + c = tail + (b + c)

            // Left side: (List.cons(head, tail) + b) + c

            // Use induction hypothesis

            // Right side: List.cons(head, tail) + (b + c)

            // Therefore
            p(List.cons(head, tail))
        }
    }
}

theorem map_singleton[T, U](f: T -> U, x: T) {
    map(List.singleton(x), f) = List.singleton(f(x))
} by {
    map[T, U](List.nil[T], f).append(f(x)) = map[T, U](List.nil[T].append(x), f)
}

theorem length_zero_imp_nil[T](list: List[T]) {
    list.length = Nat.0 implies list = List.nil[T]
} by {
    match list {
        List.nil {
            if list.length = Nat.0 {
                list = List.nil[T]
            }
        }
        List.cons(head, tail) {
            tail.length.suc != Nat.0
            if list.length = Nat.0 {
                false
            }
        }
    }
}

theorem add_to_nil[T](a: List[T], b: List[T]) {
    a + b = List.nil[T] implies a = List.nil[T] and b = List.nil[T]
} by {
    a.length + b.length = (a + b).length
    a.length + b.length != Nat.0 or b.length = Nat.0
    a.length + b.length != Nat.0 or a.length = Nat.0
    a.length != Nat.0 or List.nil[T] = a
    b.length != Nat.0 or List.nil[T] = b
}

theorem append_not_nil[T](a: List[T], t: T) {
    a.append(t) != List.nil[T]
}

theorem map_map[T, U, V](items: List[T], f: T -> U, g: U -> V) {
    map(map(items, f), g) = map(items, compose(g, f))
} by {
    define p(x: List[T]) -> Bool {
        map(map(x, f), g) = map(x, compose(g, f))
    }

    // Base case
    map[T, U](List.nil[T], f) = List.nil[U]
    map[T, V](List.nil[T], compose[T, U, V](g, f)) = List.nil[V]
    map[U, V](List.nil[U], g) = List.nil[V]
    p(List.nil)

    // Inductive step
    forall(head: T, tail: List[T]) {
        if p(tail) {
            compose[T, U, V](g, f, head) = g(f(head))
            List.cons(g(f(head)), map[U, V](map[T, U](tail, f), g)) = map[U, V](List.cons(f(head), map[T, U](tail, f)), g)
            List.cons(f(head), map[T, U](tail, f)) = map[T, U](List.cons(head, tail), f)
            List.cons(compose[T, U, V](g, f, head), map[T, V](tail, compose[T, U, V](g, f))) = map[T, V](List.cons(head, tail), compose[T, U, V](g, f))
            p(List.cons(head, tail))
        }
    }
}

/// Distributing scalar multiplication through a partial sum.
theorem partial_scalar_mul[S: Semiring](c: S, f: Nat -> S, n: Nat) {
    c * partial(f, n) = partial(mul_fn(c, f), n)
} by {
    map[S, S](map[Nat, S](n.range, f), scalar_mul(c)) = map[Nat, S](n.range, compose[Nat, S, S](scalar_mul(c), f))
    sum[S](map[S, S](map[Nat, S](n.range, f), scalar_mul(c))) = c * sum[S](map[Nat, S](n.range, f))
    forall(k: Nat) {
        compose[Nat, S, S](scalar_mul(c), f, k) = mul_fn[Nat, S](c, f, k)
    }
}

/// Shifting indices in a partial sum by adding 1.
theorem partial_shift_suc[A: AddCommMonoid](f: Nat -> A, n: Nat) {
    f(0) + partial(compose(f, Nat.suc), n) = partial(f, n.suc)
} by {
    define p(k: Nat) -> Bool {
        f(0) + partial(compose(f, Nat.suc), k) = partial(f, k.suc)
    }

    // Base case: k = 0
    sum[A](map[Nat, A](0.range, compose[Nat, Nat, A](f, Nat.suc))) = partial[A](compose[Nat, Nat, A](f, Nat.suc), 0)
    partial[A](f, 1) = f(0)
    partial[A](f, 0.suc) + A.0 = partial[A](f, 0.suc)
    map[Nat, A](List.nil[Nat], compose[Nat, Nat, A](f, Nat.suc)) = List.nil[A]
    f(0) + partial(compose(f, Nat.suc), 0) = partial(f, 0.suc)
    p(0)

    // Inductive step
    forall(k: Nat) {
        if p(k) {
            // Induction hypothesis: f(0) + partial(compose(f, Nat.suc), k) = partial(f, k.suc)

            // Expand partial(compose(f, Nat.suc), k.suc)
            map[Nat, A](k.range, compose[Nat, Nat, A](f, Nat.suc)).append(compose[Nat, Nat, A](f, Nat.suc, k)) = map[Nat, A](k.range.append(k), compose[Nat, Nat, A](f, Nat.suc))
            sum[A](map[Nat, A](k.range, compose[Nat, Nat, A](f, Nat.suc))) = partial[A](compose[Nat, Nat, A](f, Nat.suc), k)
            sum[A](map[Nat, A](k.range, compose[Nat, Nat, A](f, Nat.suc)).append(f(k.suc))) = sum[A](map[Nat, A](k.range, compose[Nat, Nat, A](f, Nat.suc))) + f(k.suc)
            compose[Nat, Nat, A](f, Nat.suc, k) = f(k.suc)
            k.range.append(k) = k.suc.range
            sum(map(k.suc.range, compose(f, Nat.suc))) = partial(compose(f, Nat.suc), k) + f(k.suc)

            // Left side

            // Apply induction hypothesis
            (f(0) + partial(compose(f, Nat.suc), k)) + f(k.suc) = partial(f, k.suc) + f(k.suc)

            // Right side: expand partial(f, k.suc.suc)
            map[Nat, A](k.suc.range, f).append(f(k.suc)) = map[Nat, A](k.suc.range.append(k.suc), f)
            sum[A](map[Nat, A](k.suc.range, f)) = partial[A](f, k.suc)
            sum[A](map[Nat, A](k.suc.range, f).append(f(k.suc))) = sum[A](map[Nat, A](k.suc.range, f)) + f(k.suc)
            k.suc.range.append(k.suc) = k.suc.suc.range
            sum(map(k.suc.suc.range, f)) = partial(f, k.suc) + f(k.suc)

            p(k.suc)
        }
    }
    forall(k: Nat) { p(k) }
}

/// Splitting off the last term of a partial sum.
theorem partial_split_last[A: AddCommMonoid](f: Nat -> A, n: Nat) {
    partial(f, n.suc) = partial(f, n) + f(n)
} by {
    // Explicate range expansion

    // Explicate map_append
    map(n.range, f).append(f(n)) = map(n.range.append(n), f)

    // Explicate sum_append

    // Explicate partial definitions
}

/// Function extensionality for partial sums: if two functions agree on all relevant indices, their partial sums are equal.
theorem partial_pointwise_eq[A: AddCommMonoid](f: Nat -> A, g: Nat -> A, n: Nat) {
    (forall(k: Nat) { k < n implies f(k) = g(k) }) implies partial(f, n) = partial(g, n)
} by {
    define p(m: Nat) -> Bool {
        (forall(k: Nat) { k < m implies f(k) = g(k) }) implies partial(f, m) = partial(g, m)
    }

    // Base case: m = 0
    sum[A](map[Nat, A](0.range, f)) = partial[A](f, 0)
    sum[A](map[Nat, A](0.range, g)) = partial[A](g, 0)
    map[Nat, A](List.nil[Nat], f) = List.nil[A]
    map[Nat, A](List.nil[Nat], g) = List.nil[A]
    p(0)

    // Inductive step
    forall(m: Nat) {
        if p(m) {
            if forall(k: Nat) { k < m.suc implies f(k) = g(k) } {
                // Split into k < m and k = m
                // Apply induction hypothesis
                forall(k: Nat) {
                    if k < m {
                        k < m.suc
                        f(k) = g(k)
                    }
                }
                partial(f, m) = partial(g, m)
                partial(f, m) + f(m) = partial(g, m) + f(m)
                // Handle k = m
                // Use partial_split_last
                partial(f, m.suc) = partial(g, m) + f(m)
                p(m.suc)
            }
            p(m.suc)
        }
    }

    p(n)
}

/// Helper: reverses the indices of a function over a range.
/// reverse_index(g, n, i) = g(n - i)
define reverse_index[A](g: Nat -> A, n: Nat, i: Nat) -> A {
    g(n - i)
}

/// Summing a function in reverse order equals summing in forward order.
theorem partial_reverse[A: AddCommMonoid](g: Nat -> A, n: Nat) {
    partial(g, n.suc) = partial(reverse_index(g, n), n.suc)
} by {
    // Prove by induction on n
    define p(m: Nat) -> Bool {
        partial(g, m.suc) = partial(reverse_index(g, m), m.suc)
    }

    // Base case: m = 0
    // partial(g, 1) = g(0)
    // partial(reverse_index(g, 0), 1) = reverse_index(g, 0, 0) = g(0 - 0) = g(0)
    partial(g, Nat.1) = g(Nat.0)
    reverse_index(g, Nat.0, Nat.0) = g(Nat.0 - Nat.0)
    Nat.0 - Nat.0 = Nat.0
    reverse_index(g, Nat.0, Nat.0) = g(Nat.0)
    partial(reverse_index(g, Nat.0), Nat.1) = reverse_index(g, Nat.0, Nat.0)
    partial(g, Nat.1) = partial(reverse_index(g, Nat.0), Nat.1)
    p(Nat.0)

    // Inductive step
    forall(m: Nat) {
        if p(m) {
            // Goal: partial(g, m.suc.suc) = partial(reverse_index(g, m.suc), m.suc.suc)
            // IH: partial(g, m.suc) = partial(reverse_index(g, m), m.suc)

            // Expand both sides using partial_split_last
            partial(g, m.suc.suc) = partial(g, m.suc) + g(m.suc)
            partial(reverse_index(g, m.suc), m.suc.suc) = partial(reverse_index(g, m.suc), m.suc) + reverse_index(g, m.suc, m.suc)

            // Evaluate reverse_index(g, m.suc, m.suc)
            reverse_index(g, m.suc, m.suc) = g(m.suc - m.suc)
            m.suc - m.suc = Nat.0
            reverse_index(g, m.suc, m.suc) = g(Nat.0)

            // So the goal becomes:
            // partial(g, m.suc) + g(m.suc) = partial(reverse_index(g, m.suc), m.suc) + g(0)

            // By IH: partial(g, m.suc) = partial(reverse_index(g, m), m.suc)
            // So: partial(reverse_index(g, m), m.suc) + g(m.suc) = partial(reverse_index(g, m.suc), m.suc) + g(0)

            // Now expand:
            // partial(reverse_index(g, m), m.suc) = g(m) + g(m-1) + ... + g(0)
            // partial(reverse_index(g, m.suc), m.suc) = g(m.suc) + g(m) + ... + g(1)
            //
            // So:
            // LHS = [g(m) + ... + g(0)] + g(m.suc) = g(m.suc) + g(m) + ... + g(0)
            // RHS = [g(m.suc) + g(m) + ... + g(1)] + g(0) = g(m.suc) + g(m) + ... + g(0)
            //
            // These are equal by commutativity of addition.

            // Use partial_drop_first to relate reverse_index(g, m.suc) to reverse_index(g, m)
            m.suc > Nat.0
            partial(reverse_index(g, m.suc), m.suc) = reverse_index(g, m.suc, Nat.0) + partial(compose(reverse_index(g, m.suc), Nat.suc), m.suc - Nat.1)
            reverse_index(g, m.suc, Nat.0) = g(m.suc - Nat.0)
            m.suc - Nat.0 = m.suc
            reverse_index(g, m.suc, Nat.0) = g(m.suc)
            m.suc - Nat.1 = m
            partial(reverse_index(g, m.suc), m.suc) = g(m.suc) + partial(compose(reverse_index(g, m.suc), Nat.suc), m)

            // Show that compose(reverse_index(g, m.suc), Nat.suc)(i) = reverse_index(g, m, i) for all i < m
            forall(i: Nat) {
                if i < m {
                    reverse_index(g, m.suc, i.suc) = g(m.suc - i.suc)
                    i <= m
                    m - i + i = m
                    m - i + i.suc = (m - i + i).suc
                    m.suc - i.suc = m - i
                    compose(reverse_index(g, m.suc), Nat.suc, i) = reverse_index(g, m, i)
                }
            }

            // Since the functions agree on all indices < m, use partial_pointwise_eq
            partial(compose(reverse_index(g, m.suc), Nat.suc), m) = partial(reverse_index(g, m), m)

            // So partial(reverse_index(g, m.suc), m.suc) = g(m.suc) + partial(reverse_index(g, m), m)

            // Use partial_split_last on partial(reverse_index(g, m), m.suc)
            m - m = Nat.0
            reverse_index(g, m, m) = g(Nat.0)

            // Now we can show:
            // partial(reverse_index(g, m), m.suc) + g(m.suc)
            // = [partial(reverse_index(g, m), m) + g(0)] + g(m.suc)
            // = partial(reverse_index(g, m), m) + g(0) + g(m.suc)
            // = partial(reverse_index(g, m), m) + g(m.suc) + g(0)   [commutativity]
            // = [g(m.suc) + partial(reverse_index(g, m), m)] + g(0)
            // = partial(reverse_index(g, m.suc), m.suc) + g(0)

            // Explicate commutativity

            // Explicate associativity
            g(m.suc) + partial(reverse_index(g, m), m) + g(Nat.0) = g(m.suc) + (partial(reverse_index(g, m), m) + g(Nat.0))


            partial[A](reverse_index[A](g, m), m.suc) = partial[A](g, m.suc)
            p(m.suc)
        }
    }
    p(n)
}

/// Extract the first term from a partial sum.
theorem partial_drop_first[A: AddCommMonoid](f: Nat -> A, n: Nat) {
    n > 0 implies partial(f, n) = f(0) + partial(compose(f, Nat.suc), n - 1)
}

/// Split a partial sum into first term, middle terms, and last term.
theorem partial_split_first_last[A: AddCommMonoid](f: Nat -> A, n: Nat) {
    n >= 2 implies partial(f, n) = f(0) + partial(compose(f, Nat.suc), n - 2) + f(n - 1)
} by {
    if n >= 2 {
        // Use partial_drop_first to extract f(0)

        // Now apply partial_split_last to partial(compose(f, Nat.suc), n - 1)
        // Need to show n - 1 >= 1 when n >= 2
        n != 0
        let (n_minus_1: Nat) satisfy { n_minus_1.suc = n }

        // Since n >= 2, we have n_minus_1 >= 1
        n_minus_1.suc >= 2
        n_minus_1 >= 1

        let (n_minus_2: Nat) satisfy { n_minus_2.suc = n_minus_1 }

        // Explicate n - 2 = n_minus_2
        n_minus_2 + 2 = n
        n - 2 = n_minus_2

        // Explicate partial_shift_suc

        // Explicate partial_split_last
        partial(f, n_minus_1) + f(n_minus_1) = partial(f, n_minus_1.suc)

        // Explicate suc_sub_one

        partial(f, n) = f(0) + partial(compose(f, Nat.suc), n - 2) + f(n - 1)
    }
}

theorem map_length[T, U](list: List[T], f: T -> U) {
    map(list, f).length = list.length
} by {
    define p(x: List[T]) -> Bool {
        map(x, f).length = x.length
    }

    // Base case
    p(List.nil[T])

    // Inductive step
    forall(head: T, tail: List[T]) {
        if p(tail) {
            map[T, U](tail, f).length = tail.length
            List.cons(f(head), map[T, U](tail, f)).length = map[T, U](tail, f).length.suc
            p(List.cons(head, tail))
        }
    }

    forall(l: List[T]) { p(l) }
}

theorem map_contains[T, U](list: List[T], f: T -> U, item: U) {
    map(list, f).contains(item) implies exists(x: T) {
        list.contains(x) and f(x) = item
    }
} by {
    define p(x: List[T]) -> Bool {
        map(x, f).contains(item) implies exists(y: T) {
            x.contains(y) and f(y) = item
        }
    }

    // Base case
    map[T, U](List.nil[T], f) = List.nil[U]
    not List.nil[U].contains(item)
    p(List.nil[T])

    // Inductive step
    forall(head: T, tail: List[T]) {
        if p(tail) {
            if map(List.cons(head, tail), f).contains(item) {
                // If the map contains the item, it must be from either head or tail
                if map(tail, f).contains(item) {
                    // If tail contains the item, then we can use the induction hypothesis
                } else {
                    // If head is the source of the item
                    List.cons(f(head), map[T, U](tail, f)) = map[T, U](List.cons(head, tail), f)
                    f(head) = item
                    p(List.cons(head, tail))
                }
            }
            List.nil[T] + tail = tail
            List.cons(head, List.nil[T]) + tail = List.cons(head, List.nil[T] + tail)
            List.cons(head, List.nil[T]) + tail = List.cons(head, tail)
            forall(x: T) { tail.contains(x) implies (List.cons(head, List.nil[T]) + tail).contains(x) }
            if map(List.cons(head, List.nil[T]) + tail, f).contains(item) {
                if map(tail, f).contains(item) {
                    let x: T satisfy {
                        tail.contains(x) and f(x) = item
                    }
                    exists(y: T) {
                        (List.cons(head, List.nil[T]) + tail).contains(y) and f(y) = item
                    }
                } else {
                    List.cons(f(head), map[T, U](tail, f)) = map[T, U](List.cons(head, tail), f)
                    map[T, U](List.cons(head, tail), f) = map[T, U](List.cons(head, List.nil[T]) + tail, f)
                    f(head) = item
                    exists(y: T) {
                        (List.cons(head, List.nil[T]) + tail).contains(y) and f(y) = item
                    }
                }
                p(List.cons(head, List.nil[T]) + tail)
            }
            p(List.cons(head, List.nil[T]) + tail)
            p(List.cons(head, tail))
        }
    }

}

/// Membership is preserved by mapping a function over a list.
theorem map_contains_of_contains[T, U](list: List[T], f: T -> U, item: T) {
    list.contains(item) implies map(list, f).contains(f(item))
} by {
    define p(x: List[T]) -> Bool {
        x.contains(item) implies map(x, f).contains(f(item))
    }

    p(List.nil[T])

    forall(head: T, tail: List[T]) {
        if p(tail) {
            if List.cons(head, tail).contains(item) {
                if head = item {
                    List.cons(f(head), map[T, U](tail, f)).contains(f(item))
                    map(List.cons(head, tail), f).contains(f(item))
                } else {
                    map(tail, f).contains(f(item))
                    List.cons(f(head), map[T, U](tail, f)).contains(f(item))
                    map(List.cons(head, tail), f).contains(f(item))
                }
            }
            p(List.cons(head, tail))
        }
    }
}

theorem pigeonhole_unique_map[T, U](items: List[T], f: T -> U) {
    items.is_unique and not map(items, f).is_unique implies
    exists(x: T, y: T) {
        x != y and f(x) = f(y)
    }
} by {
    define p(l: List[T]) -> Bool {
        l.is_unique and not map(l, f).is_unique implies
        exists(a: T, b: T) {
            a != b and f(a) = f(b)
        }
    }

    define has_duplicate(list: List[U]) -> Bool {
        exists(y: U) {
            list.count(y) > 1
        }
    }

    // Base case: empty list
    map[T, U](List.nil[T], f) = List.nil[U]
    List.nil[U].unique = List.nil[U]
    List.nil[U].is_unique
    p(List.nil[T])

    // Inductive step
    forall(head: T, tail: List[T]) {
        if p(tail) {
            if List.cons(head, tail).is_unique and not map(List.cons(head, tail), f).is_unique {
                let tail_map = map(tail, f)
                // Either tail already has duplicate or not
                if has_duplicate(tail_map) {
                    let dup: U satisfy {
                        tail_map.count(dup) > 1
                    }
                    if map(tail, f).is_unique {
                        map(tail, f).count(dup) <= Nat.1
                        false
                    }
                    not map(tail, f).is_unique

                } else {
                    // If tail_map does not have a duplicate, f(head) is the
                    // duplicate
                    let dup = f(head)

                    // Explicate: tail_map is unique since it has no duplicate
                    tail_map.is_unique
                    tail_map.unique = tail_map

                    // Explicate: map(cons(head, tail), f) equals cons(f(head), tail_map)
                    List.cons(f(head), map(tail, f)) = map(List.cons(head, tail), f)

                    // Since map(List.cons(head, tail), f) is not unique, its unique is different
                    map(List.cons(head, tail), f).unique != map(List.cons(head, tail), f)

                    // This means dup is in tail_map
                    tail_map.contains(dup)

                    // Explicate: dup is in map(tail, f)
                    map(tail, f).contains(dup)

                    let x: T satisfy {
                        tail.contains(x) and f(x) = dup
                    }

                    // Explicate: uniqueness facts
                    tail.is_unique
                    tail.unique = tail

                    // Explicate: head not in tail since cons(head, tail) is unique
                    List.cons(head, tail).unique = List.cons(head, tail)

                    // x != head since x is in tail but head is not in tail
                    x != head

                    // We have f(x) = f(head) and x != head

                    p(List.cons(head, tail))
                }

                p(List.cons(head, tail))
            }
            p(List.cons(head, tail))
        }
    }

    p(items)
}

theorem pigeonhole_map[U, T](items: List[T], f: T -> U) {
    items.is_unique and items.length > map(items, f).unique.length implies
    exists(x: T, y: T) {
        x != y and f(x) = f(y)
    }
} by {
    not map(items, f).is_unique
}

/// The unique image list is no longer than any list containing the image.
theorem map_unique_length_le_of_contains[T, U](items: List[T], targets: List[U], f: T -> U) {
    forall(x: T) {
        items.contains(x) implies targets.contains(f(x))
    } implies map(items, f).unique.length <= targets.length
} by {
    if forall(x: T) {
        items.contains(x) implies targets.contains(f(x))
    } {
        forall(y: U) {
            if map(items, f).contains(y) {
                map_contains(items, f, y)
                let x: T satisfy {
                    items.contains(x) and f(x) = y
                }
                targets.contains(y)
            }
        }
        unique_is_smallest_containing_list(map(items, f), targets)
        map(items, f).unique.length <= targets.length
    }
}

/// Mapping a unique list into a shorter containing list forces a collision.
theorem pigeonhole_map_into_list[T, U](items: List[T], targets: List[U], f: T -> U) {
    items.is_unique and items.length > targets.length and
    forall(x: T) {
        items.contains(x) implies targets.contains(f(x))
    } implies exists(x: T, y: T) {
        x != y and f(x) = f(y)
    }
} by {
    map_unique_length_le_of_contains(items, targets, f)
    map(items, f).unique.length <= targets.length
    map(items, f).unique.length < items.length
    pigeonhole_map(items, f)
}

/// Mapping an injective function over a unique list gives a unique list.
theorem injective_map_is_unique[T, U](items: List[T], f: T -> U) {
    items.is_unique and is_injective_fn(f) implies map(items, f).is_unique
} by {
    if items.is_unique and is_injective_fn(f) {
        if not map(items, f).is_unique {
            pigeonhole_unique_map(items, f)
            let (x: T, y: T) satisfy {
                x != y and f(x) = f(y)
            }
            injective_fn_eq(f, x, y)
            false
        }
        map(items, f).is_unique
    }
}

attributes List[T] {
    /// Yields the list without its first element.
    /// Yields nil for an empty list.
    define tail(self) -> List[T] {
        match self {
            List.nil {
                List.nil[T]
            }
            List.cons(h, t) {
                t
            }
        }
    }

    /// Removes the first n elements from the list.
    define drop(self, n: Nat) -> List[T] {
        match n {
            Nat.zero {
                self
            }
            Nat.suc(pred) {
                self.tail.drop(pred)
            }
        }
    }
}

theorem tail_cancels_cons[T](a: T, b: List[T]) {
    List.cons(a, b).tail = b
}

theorem alt_drop_zero[T](a: List[T], n: Nat) {
    n = Nat.0 implies a.drop(n) = a
}

theorem drop_zero[T](a: List[T]) {
    a.drop(Nat.0) = a
}

theorem drop_one[T](a: List[T]) {
    a.drop(Nat.1) = a.tail
}

theorem drop_cancels_add[T](a: List[T], b: List[T]) {
    (a + b).drop(a.length) = b
} by {
    define p(x: List[T]) -> Bool {
        (x + b).drop(x.length) = b
    }

    // Base case
    p(List.nil)

    // Inductive step
    forall(head: T, tail: List[T]) {
        if p(tail) {
            // Induction hypothesis: (tail + b).drop(tail.length) = b

            // Left side

            // Use induction hypothesis
            List.cons(head, tail) + b = List.cons(head, tail + b)
            List.cons(head, tail).length = tail.length.suc
            List.cons(head, tail + b).tail = tail + b
            List.cons(head, tail + b).tail.drop(tail.length) = List.cons(head, tail + b).drop(tail.length.suc)
            // Therefore
            p(List.cons(head, tail))
        }
    }
}

theorem drop_twice_all[T](m: Nat) {
    forall(l: List[T], k: Nat) {
        l.drop(m).drop(k) = l.drop(m + k)
    }
} by {
    define f(x: Nat) -> Bool {
        forall(l: List[T], k: Nat) {
            l.drop(x).drop(k) = l.drop(x + k)
        }
    }

    f(Nat.0)

    forall(x: Nat) {
        if f(x) {
            forall(l: List[T], k: Nat) {
                // Induction hypothesis: l.drop(x).drop(k) = l.drop(x + k)
                l.drop(x.suc).drop(k) = l.drop(x.suc + k)
            }
            f(x.suc)
        }
    }
    f(m)
}

theorem drop_twice[T](a: List[T], m: Nat, n: Nat) {
    a.drop(m).drop(n) = a.drop(m + n)
} by {
}

attributes List[T] {
    /// Removes the last n elements from the list.
    define drop_last(self, n: Nat) -> List[T] {
        match self {
            List.nil {
                List.nil[T]
            }
            List.cons(head, tail) {
                if tail.length < n {
                    List.nil[T]
                } else {
                    List.cons(head, tail.drop_last(n))
                }
            }
        }
    }
}

theorem drop_last_zero[T](a: List[T]) {
    a.drop_last(Nat.0) = a
} by {
    define q(x: List[T]) -> Bool {
        x.drop_last(Nat.0) = x
    }
    q(List.nil[T])
    forall(head: T, tail: List[T]) {
        if q(tail) {
            not tail.length < Nat.0
            List.cons(head, tail).drop_last(Nat.0) = List.cons(head, tail.drop_last(Nat.0)) or tail.length < Nat.0
            q(List.cons(head, tail))
        }
    }
    forall(x: List[T]) {
        q(x)
    }
}

theorem drop_last_all[T](l: List[T]) {
    l.drop_last(l.length) = List.nil[T]
} by {
    define r(x: List[T]) -> Bool {
        x.drop_last(x.length) = List.nil[T]
    }
    r(List.nil[T])
    forall(head: T, tail: List[T]) {
        if r(tail) {
            r(List.cons(head, tail))
        }
    }
    forall(x: List[T]) {
        r(x)
    }
}

theorem drop_last_cancels_add[T](a: List[T], b: List[T]) {
    (a + b).drop_last(b.length) = a
} by {
    define p(x: List[T]) -> Bool {
        (x + b).drop_last(b.length) = x
    }
    p(List.nil[T])
    forall(head: T, tail: List[T]) {
        if p(tail) {
            tail.length + b.length = (tail + b).length
            tail.length + b.length = b.length + tail.length
            b.length <= b.length + tail.length
            not b.length <= b.length + tail.length or not b.length + tail.length < b.length
            List.cons(head, tail + b).drop_last(b.length) = List.cons(head, (tail + b).drop_last(b.length))
            p(List.cons(head, tail))
        }
    }
    forall(x: List[T]) {
        p(x)
    }
}

attributes Nat {
    /// Creates a list of natural numbers from self to n-1 (exclusive of n).
    define until(self, n: Nat) -> List[Nat] {
        n.range.drop(self)
    }

    /// Creates a list of natural numbers from self to n (inclusive).
    define upto(self, n: Nat) -> List[Nat] {
        self.until(n.suc)
    }
}

theorem zero_until(n: Nat) {
    Nat.0.until(n) = n.range
} by {
}

theorem until_self(n: Nat) {
    n.until(n) = List.nil[Nat]
} by {
    (n.range + List.nil[Nat]).drop(n.range.length) = List.nil[Nat]
}

theorem until_suc(n: Nat) {
    n.until(n.suc) = List.singleton(n)
} by {
    (n.range + List.singleton(n)).drop(n.range.length) = List.singleton(n)
}

theorem zero_upto(n: Nat) {
    Nat.0.upto(n) = n.suc.range
}

theorem upto_self(n: Nat) {
    n.upto(n) = List.singleton(n)
}

theorem range_add_until(a: Nat, b: Nat) {
    a <= b implies a.range + a.until(b) = b.range
} by {
    let (k: Nat) satisfy { a + k = b }

    define f(x: Nat) -> Bool {
        a.range + a.until(a + x) = (a + x).range
    }

    // Base case: x = 0
    a.until(a) = List.nil[Nat]
    f(Nat.0)

    // Inductive step
    forall(x: Nat) {
        if f(x) {
            // Show a.until(a + x.suc) = a.until(a + x) + List.singleton(a + x)
            (a.range + (a.until(a + x) + List.singleton(a + x))).drop(a.range.length) = a.until(a + x) + List.singleton(a + x)

            // Now show f(x.suc)
            a + x.suc = (a + x).suc
            a.range + a.until(a + x) + List.singleton(a + x) = (a.range + a.until(a + x)).append(a + x)
            (a + x.suc).range.drop(a) = a.until(a + x.suc)
            a.range.length = a
            (a + x).range.append(a + x) = (a + x).suc.range

            // Explicate: Use the induction hypothesis
            a.range + a.until(a + x) = (a + x).range

            f(x.suc)
        }
    }

    f(k)
}

// Could be generalized to arbitrary functions over a set with 0, not
// necessarily associative as in `add`. `LinearOrder` with smallest element 0
// would similarly work (or with `option`). Technically `Nat`s w/ `max` defines a
// `Monoid`, which would also work w/ definition of `add` above.
define max_list(list: List[Nat]) -> Nat {
    match list {
        List[Nat].nil {
            0
        }
        List.cons(head, tail) {
            head.max(max_list(tail))
        }
    }
}

theorem list_has_max(list: List[Nat], n: Nat) {
    max_list(list) <= n implies
    forall(m: Nat) {
        list.contains(m) implies m <= n
    }
} by {
    define f(l: List[Nat]) -> Bool {
        forall(k: Nat) {
            l.contains(k) implies k <= max_list(l)
        }
    }

    not List.nil[Nat].contains(Nat.0)
    f(List.nil[Nat])

    forall(head: Nat, tail: List[Nat]) {
        if f(tail) {
            let k = max_list(tail)
            let m: Nat = head.max(k)
            m >= max_list(tail)
            forall(x: Nat) {
                tail.contains(x) implies x <= m

                List.cons(head, tail).contains(x) implies x <= m
            }

            m = max_list(List.cons(head, tail))

            forall(i: Nat) {
                if List.cons(head, tail).contains(i) {
                    i <= max_list(List.cons(head, tail))
                }
            }
            forall(i: Nat) {
                List.cons(head, tail).contains(i) implies i <= max_list(List.cons(head, tail))
            }

            f(List.cons(head, tail))
        }
    }

    f(list)
}

theorem no_list_contains_nat(list: List[Nat]) {
    exists(n: Nat) {
        not list.contains(n)
    }
}

attributes List[T] {
    /// The index of the first occurrence of the item in the list.
    /// Returns the list length if the item is not found.
    define find_first_idx(self, item: T) -> Nat {
        match self {
            List.nil {
                Nat.0
            }
            List.cons(head, tail) {
                if head = item {
                    Nat.0
                } else {
                    1 + tail.find_first_idx(item)
                }
            }
        }
    }

    /// The element at index i, or none if the index is out of bounds.
    define get_idx(self, i: Nat) -> Option[T] {
        match self {
            List.nil {
                Option.none
            }
            List.cons(head, tail) {
                if i > 0 {
                    tail.get_idx(i - 1)
                } else {
                    Option.some(head)
                }
            }
        }
    }
}

theorem find_first_idx_contains[T](list: List[T], item: T) {
    list.contains(item) implies list.find_first_idx(item) < list.length
} by {
    define p(l: List[T]) -> Bool {
        l.contains(item) implies l.find_first_idx(item) < l.length
    }

    // Base case: empty list does not contain any item
    not List.nil[T].contains(item)
    p(List.nil[T])

    // Inductive step
    forall(head: T, tail: List[T]) {
        if p(tail) {
            if List.cons(head, tail).contains(item) {
                if head = item {
                    p(List.cons(head, tail))
                } else {
                    tail.contains(item)
                    tail.find_first_idx(item) < tail.length
                    Nat.1 + tail.find_first_idx(item) = tail.find_first_idx(item).suc
                    List.cons(head, tail).length = tail.length.suc
                    not tail.length <= tail.find_first_idx(item) or not tail.find_first_idx(item) < tail.length
                    tail.length.suc <= tail.find_first_idx(item).suc or tail.find_first_idx(item).suc < tail.length.suc
                    not tail.length.suc <= tail.find_first_idx(item).suc or tail.length <= tail.find_first_idx(item)
                    p(List.cons(head, tail))
                }
            }
            p(List.cons(head, tail))
        }
    }

    forall(l: List[T]) { p(l) }
}

theorem find_first_idx_get_idx[T](list: List[T], item: T) {
    list.contains(item) implies
    list.get_idx(list.find_first_idx(item)) = Option.some(item)
} by {
    define p(l: List[T]) -> Bool {
        l.contains(item) implies
        l.get_idx(l.find_first_idx(item)) = Option.some(item)
    }

    // Base case: empty list does not contain any item
    not List.nil[T].contains(item)
    p(List.nil[T])

    // Inductive step
    forall(head: T, tail: List[T]) {
        if p(tail) {
            if List.cons(head, tail).contains(item) {
                if head = item {
                    // If head is the item, then find_first_idx returns 0 and get_idx(0) returns head
                    List.cons(head, tail).find_first_idx(item) = 0
                    List.cons(head, tail).get_idx(0) = Option.some(head)

                    p(List.cons(head, tail))
                } else {
                    // If not, then find_first_idx returns the index in tail and get_idx returns the item
                    let idx = tail.find_first_idx(item)

                    // Explicate: 1 + idx = idx.suc
                    idx.suc > 0

                    // Explicate: get_idx cons rule for positive index
                    tail.get_idx(idx.suc - 1) = List.cons(head, tail).get_idx(idx.suc)

                    List.cons(head, tail).get_idx(1 + idx) = tail.get_idx(idx)

                    // Explicate: item is in tail since head != item
                    tail.contains(item)

                    // Explicate: find_first_idx for cons when head != item
                    List.cons(head, tail).find_first_idx(item) = 1 + tail.find_first_idx(item)

                    // Explicate: induction hypothesis
                    tail.get_idx(tail.find_first_idx(item)) = Option.some(item)

                    p(List.cons(head, tail))
                }
            }
            p(List.cons(head, tail))
        }
    }

    forall(l: List[T]) { p(l) }
}

theorem get_idx_succ_implies_tail[T](list: List[T], i: Nat) {
    i + 1 < list.length implies list.get_idx(i + 1) = list.tail.get_idx(i)
} by {
    define p(l: List[T]) -> Bool {
        i + 1 < l.length implies l.get_idx(i + 1) = l.tail.get_idx(i)
    }

    // Base case: empty list does not have any valid index
    p(List.nil[T])

    // Inductive step
    forall(head: T, tail: List[T]) {
        if p(tail) {
            if i + 1 < List.cons(head, tail).length {
                i + Nat.1 = i.suc
                i.suc != Nat.0
                not i.suc < Nat.0
                i + Nat.1 - Nat.1 = i
                (i + Nat.1 > Nat.0) = (Nat.0 < i + Nat.1)
                not i + Nat.1 > Nat.0 or List.cons(head, tail).get_idx(i + Nat.1) = tail.get_idx(i + Nat.1 - Nat.1)
                List.cons(head, tail).get_idx(i + 1) = tail.get_idx(i)

                p(List.cons(head, tail))
            }
            p(List.cons(head, tail))
        }
    }

    p(list)
}

theorem duplicate_implies_duplicate_idx[T](list: List[T], item: T) {
    list.count(item) > 1 implies
    exists(i: Nat, j: Nat) {
        i < j and j < list.length and
        list.get_idx(i) = Option.some(item) and
        list.get_idx(j) = Option.some(item)
    }
} by {
    define p(l: List[T]) -> Bool {
        l.count(item) > 1 implies
        exists(i: Nat, j: Nat) {
            i < j and j < l.length and
            l.get_idx(i) = Option.some(item) and
            l.get_idx(j) = Option.some(item)
        }
    }

    // Base case: empty list does not contain any item
    List.nil[T].count(item) = Nat.0
    not Nat.1 < Nat.0
    p(List.nil[T])

    // Inductive step
    forall(head: T, tail: List[T]) {
        if p(tail) {
            if List.cons(head, tail).count(item) > 1 {
                if head = item {
                    // If head is the item, then we need one more in the tail
                    Nat.1 + tail.count(item) = List.cons(item, tail).count(item)
                    List.cons(head, tail).count(item) > 1 = (Nat.1 < List.cons(head, tail).count(item))
                    not Nat.1 < List.cons(head, tail).count(item) or List.cons(head, tail).count(item) != Nat.1
                    Nat.1 + tail.count(item) = tail.count(item).suc
                    tail.count(item) = Nat.0 or tail.contains(item)
                    tail.contains(item)
                    let idx = tail.find_first_idx(item)
                    idx + 1 < tail.length + 1

                    // Explicate: idx + 1 > 0
                    idx + 1 > 0
                    idx + 1 - 1 = idx

                    List.cons(head, tail).get_idx(idx + 1) = tail.get_idx(idx)

                    // Indices
                    let i = 0
                    let j = idx + 1
                    i < j
                    List.cons(head, tail).get_idx(i) = Option.some(head)
                    tail.length + Nat.1 = tail.length.suc
                    not tail.contains(item) or tail.get_idx(tail.find_first_idx(item)) = Option.some(item)
                    j < List.cons(head, tail).length
                    List.cons(head, tail).get_idx(j) = Option.some(item)

                    p(List.cons(head, tail))
                } else {
                    // If head is not the item, then we need two in the tail
                    List.cons(head, tail).count(item) = tail.count(item)
                    tail.count(item) > 1

                    let (i: Nat, j: Nat) satisfy {
                        i < j and j < tail.length and
                        tail.get_idx(i) = Option.some(item) and
                        tail.get_idx(j) = Option.some(item)
                    }

                    i + 1 > 0
                    List.cons(head, tail).get_idx(i + 1) = tail.get_idx(i)
                    j + 1 > 0
                    List.cons(head, tail).get_idx(j + 1) = tail.get_idx(j)

                    i + 1 < j + 1
                    tail.length.suc <= j.suc or j.suc < tail.length.suc
                    not tail.length <= j or not j < tail.length
                    not tail.length.suc <= j.suc or tail.length <= j

                    // Explicate: indices and values for p
                    List.cons(head, tail).get_idx(j + 1) = Option.some(item)
                    j + 1 < List.cons(head, tail).length

                    p(List.cons(head, tail))
                }
            }
            p(List.cons(head, tail))
        }
    }

    p(list)
}

theorem index_pigeonhole[T](list: List[T]) {
    list.unique.length < list.length implies
    exists(i: Nat, j: Nat) {
        i < j and j < list.length and
        list.get_idx(i) = list.get_idx(j)
    }
} by {
    // Explicit: state the theorem

    if list.unique.length < list.length {
        // unique_len_smaller_implies_duplicate gives us a duplicate item
        let item: T satisfy { list.count(item) > 1 }

        // Explicit: state the second theorem for this item

        // duplicate_implies_duplicate_idx gives us two indices for that item
        let (i: Nat, j: Nat) satisfy {
            i < j and j < list.length and
            list.get_idx(i) = Option.some(item) and
            list.get_idx(j) = Option.some(item)
        }

        exists(i2: Nat, j2: Nat) {
            i2 < j2 and j2 < list.length and
            list.get_idx(i2) = list.get_idx(j2)
        }
    }
}

theorem get_idx_always_some[T](list: List[T], idx: Nat) {
    idx < list.length implies exists(x: T) {
        list.get_idx(idx) = Option.some(x)
    }
} by {
    define pl(i: Nat, l: List[T]) -> Bool {
        i < l.length implies exists(x: T) {
            l.get_idx(i) = Option.some(x)
        }
    }

    define p(i: Nat) -> Bool {
        forall(l: List[T]) {
            pl(i, l)
        }
    }

    forall(l: List[T]) {
        match l {
            List.nil {
            }
            List.cons(head, tail) {
                List.cons(head, tail) = l
                l.get_idx(0) = Option.some(head)
                pl(0, l)
            }
        }
    }

    forall(l: List[T]) {
        pl(0, l)
    }
    p(0)

    forall(i: Nat) {
        if p(i) {
            forall(l: List[T]) {
                if i + 1 >= l.length {
                    // Trivial
                } else {
                    i + 1 < l.length
                    match l {
                        List.nil {
                        }
                        List.cons(head, tail) {
                            let x: T satisfy {
                                tail.get_idx(i) = Option.some(x)
                            }
                            not i + Nat.1 < l.length or l.get_idx(i + Nat.1) = l.tail.get_idx(i)
                            List.cons(head, tail) != l or l.tail = tail
                            pl(i+1, l)
                        }
                    }
                }
                i + Nat.1 < l.length or pl(i + Nat.1, l)
                (l.length <= i + Nat.1) = (i + Nat.1 >= l.length)
                not l.length <= i + Nat.1 or not i + Nat.1 < l.length
                i + Nat.1 = Nat.1 + i
                Nat.1 + i = i.suc
                i.suc - Nat.0 = i.suc
                not i + Nat.1 < l.length or i + Nat.1 != l.length
                pl(i+1, l)
            }
            forall(l: List[T]) {
                pl(i+1, l)
            }
            p(i+1)
        }
    }

    // Apply induction to get p(idx)
    p(idx)
    pl(idx, list)
}

// This is pretty clunky unfortunately
// Might indicate that Option.some isn't great (maybe we return
// a silly object if not) or declare some monoid "lift" for
// option
/// Theorem: map(a, f).get_idx(idx) = f(a.get_idx(idx)) if idx < a.length
theorem map_under_idx[T, U](a: List[T], f: T -> U, idx: Nat) {
    idx < a.length implies exists(x: T) {
        Option.some(x) = a.get_idx(idx) and
        map(a, f).get_idx(idx) = Option.some(f(x))
    }
} by {
    define pf(i: Nat, l: List[T]) -> Bool {
        i < l.length implies exists(x: T) {
            Option.some(x) = l.get_idx(i) and
            map(l, f).get_idx(i) = Option.some(f(x))
        }
    }

    define p(i: Nat) -> Bool {
        forall(l: List[T]) {
            pf(i, l)
        }
    }

    forall(l: List[T]) {
        match l {
            List.nil {
            }
            List.cons(head, tail) {
                map(l, f).get_idx(0) = Option.some(f(head))
                pf(0, l)
            }
        }
    }

    forall(l: List[T]) {
        pf(0, l)
    }
    p(0)

    forall(i: Nat) {
        if p(i) {
            forall(l: List[T]) {
                if i + 1 >= l.length {
                    // Trivial
                    pf(i+1, l)
                } else {
                    match l {
                        List.nil {
                        }
                        List.cons(head, tail) {
                            // Explicate: i + 1 = i.suc > 0
                            i + 1 = i.suc
                            i.suc != 0
                            i + 1 > 0
                            i + 1 - 1 = i

                            // Explicate: get_idx rule for cons
                            tail.get_idx(i + 1 - 1) = l.get_idx(i + 1)

                            l.get_idx(i + 1) = tail.get_idx(i)
                            i < tail.length

                            // Explicate: pf(i, tail) from p(i)
                            pf(i, tail)
                            i < tail.length implies exists(x: T) {
                                Option.some(x) = tail.get_idx(i) and
                                map(tail, f).get_idx(i) = Option.some(f(x))
                            }

                            let x: T satisfy {
                                Option.some(x) = tail.get_idx(i) and
                                map(tail, f).get_idx(i) = Option.some(f(x))
                            }
                            List.cons(f(head), map[T, U](tail, f)) = map[T, U](List.cons(head, tail), f)
                            map(List.cons(head, tail), f).get_idx(i + 1) = map(tail, f).get_idx(i + 1 - 1)
                            map(l, f).get_idx(i + Nat.1) = Option.some(f(x))

                            pf(i+1, l)
                        }
                    }
                    pf(i+1, l)
                }
            }
        }
    }

    forall(i: Nat) { p(i) implies p(i.suc) }
    p(idx)
    pf(idx, a)
}

theorem append_add_idx_left[T](a: List[T], b: List[T], n: Nat) {
    n < a.length implies (a + b).get_idx(n) = a.get_idx(n)
} by {
    define fp(i: Nat, l: List[T]) -> Bool {
        i < l.length implies (l + b).get_idx(i) = l.get_idx(i)
    }

    define f(i: Nat) -> Bool {
        forall(l: List[T]) {
            fp(i, l)
        }
    }

    forall(l: List[T]) {
        match l {
            List.nil {
            }
            List.cons(head, tail) {
                (l + b).get_idx(0) = Option.some(head)
                fp(0, l)
            }
        }
    }

    forall(l: List[T]) {
        fp(0, l)
    }
    f(0)

    forall(i: Nat) {
        if f(i) {
            forall(l: List[T]) {
                if i+1 < l.length {
                    match l {
                        List.nil {
                        }
                        List.cons(head, tail) {
                            // Explicate: i + 1 and length facts
                            i + 1 = i.suc
                            List.cons(head, tail).length = tail.length.suc
                            i < tail.length
                            l.tail = tail

                            // Explicate: fp from f
                            fp(i, tail)
                            (tail + b).get_idx(i) = tail.get_idx(i)

                            List.cons(head, tail).get_idx(i + 1) = (tail + b).get_idx(i)
                            i + 1 > 0
                            List.cons(head, tail + b).get_idx(i + 1) = (tail + b).get_idx(i)
                            fp(i+1, l)
                        }
                    }
                    fp(i+1, l)
                }
            }
            forall(l: List[T]) {
                fp(i+1, l)
            }
            f(i+1)
        }
    }

    // Apply induction to get f(n)
    f(n)
    fp(n, a)
}

theorem append_add_singleton_right[T](list: List[T], a: T) {
    (list + List.singleton(a)).get_idx(list.length) = Option.some(a)
} by {
    define f(l: List[T]) -> Bool {
        (l + List.singleton(a)).get_idx(l.length) = Option.some(a)
    }

    // Base case: nil + singleton(a) = singleton(a), and get_idx(0) = some(a)
    List.cons(a, List.nil[T]) = List.singleton(a)
    List.nil[T] + List.singleton(a) = List.singleton(a)
    List.nil[T].length = 0
    List.singleton(a).get_idx(0) = Option.some(a)
    f(List.nil[T])

    forall(head: T, tail: List[T]) {
        if f(tail) {
            let l = List.cons(head, tail)
            let sa = List.singleton(a)

            // Explicit length facts
            l.length = tail.length.suc
            tail.length.suc - 1 = tail.length
            l.length - 1 = tail.length

            // IH: f(tail) means (tail + sa).get_idx(tail.length) = Option.some(a)
            (tail + sa).get_idx(tail.length) = Option.some(a)
            (l + sa).get_idx(l.length) = (tail + sa).get_idx(l.length - 1)

            f(List.cons(head, tail))
        }
    }
    forall(l: List[T]) {
        f(l)
    }
}

theorem range_idx_eq_idx(n: Nat, idx: Nat) {
    idx < n implies n.range.get_idx(idx) = Option.some(idx)
} by {
    define f(m: Nat) -> Bool {
        idx < m implies m.range.get_idx(idx) = Option.some(idx)
    }

    f(0)

    forall(m: Nat) {
        if f(m) {
            if idx < m + 1 {
                if idx < m {
                    (m.range + List.singleton(m)).get_idx(idx) = m.range.get_idx(idx)
                    f(m + 1)
                } else {
                    (m.range + List.singleton(m)).get_idx(m) = Option.some(m)
                    f(m + 1)
                }
            }
            f(m + 1)
        }
    }

    // Apply induction to get f(n)
    f(n)
}

theorem map_range[T](n: Nat, idx: Nat, f: Nat -> T) {
    idx < n implies map(n.range, f).get_idx(idx) = Option.some(f(idx))
} by {

    // Prover help
    if idx < n {
        let x: Nat satisfy {
            Option.some(x) = n.range.get_idx(idx) and
            map(n.range, f).get_idx(idx) = Option.some(f(x))
        }
        Option.some(x) = Option.some(idx)
        map(n.range, f).get_idx(idx) = Option.some(f(idx))
    }

}

/// Given a function `f: Nat -> T`, if there is some `n` such that
/// `map(n.range, f).unique.length < n` (in other words, at least one element in
/// the map is repeated), then there exist indices `i < j` such that
/// `f(i) = f(j)`.
theorem range_pigeonhole[T](n: Nat, f: Nat -> T) {
    map(n.range, f).unique.length < n implies exists (i: Nat, j: Nat) {
        i < j and j < n and f(i) = f(j)
    }
} by {
    let f_out = map(n.range, f)
    f_out.length = n.range.length
    n.range.length = n
    f_out.length = n
    f_out.unique.length < f_out.length

    let (i: Nat, j: Nat) satisfy {
        i < j and j < f_out.length and f_out.get_idx(i) = f_out.get_idx(j)
    }
    j < n

    map(n.range, f).get_idx(i) = Option.some(f(i))

    // Explicate: map_range for j
    map(n.range, f).get_idx(j) = Option.some(f(j))

    // Explicate: Option equality
    Option.some(f(j)) = Option.some(f(i)) implies f(j) = f(i)

    f(i) = f(j)
}

/// If an index is greater than or equal to the list length, get_idx returns none.
theorem get_idx_out_of_bounds[T](list: List[T], idx: Nat) {
    idx >= list.length implies list.get_idx(idx) = Option.none[T]
} by {
    define p(l: List[T]) -> Bool {
        forall(i: Nat) {
            i >= l.length implies l.get_idx(i) = Option.none[T]
        }
    }

    // Base case: nil
    p(List.nil[T])

    // Inductive case
    forall(head: T, tail: List[T]) {
        if p(tail) {
            forall(i: Nat) {
                if i >= List.cons(head, tail).length {
                    // cons.length = tail.length + 1

                    if i = 0 {
                        // i = 0 but i >= tail.length.suc >= 1, contradiction
                    } else {
                        // i > 0, so cons.get_idx(i) = tail.get_idx(i-1)
                        let (i_pred: Nat) satisfy { i_pred.suc = i }

                        // Explicate: i > 0 and i - 1 = i_pred
                        i > 0
                        i_pred.suc - 1 = i_pred
                        i - 1 = i_pred

                        List.cons(head, tail).get_idx(i) = tail.get_idx(i_pred)

                        // i >= tail.length.suc means i_pred.suc >= tail.length.suc
                        // so i_pred >= tail.length

                        // By induction hypothesis
                        List.cons(head, tail).get_idx(i) = Option.none[T]
                    }
                }
            }
            List.cons(head, tail).length = tail.length.suc
            tail.length.suc != Nat.0
            not tail.length.suc <= Nat.0 or tail.length.suc = Nat.0
            List.cons(head, tail).get_idx(Nat.0) = Option.some(head)
            List.cons(head, tail).get_idx(Nat.0) != Option.none[T]
            forall(i: Nat) { i >= List.cons(head, tail).length implies List.cons(head, tail).get_idx(i) = Option.none[T] }
            p(List.cons(head, tail))
        }
    }

    p(list)
}

/// Helper predicate: two lists differ at some index less than n.
define differ[T](a: List[T], b: List[T], n: Nat) -> Bool {
    exists(i: Nat) {
        i < n and a.get_idx(i) != b.get_idx(i)
    }
}

/// If two lists are different and both have length at most n,
/// then there is an index i < n where they differ.
theorem lists_differ_at_index[T](a: List[T], b: List[T], n: Nat) {
    a != b and a.length <= n and b.length <= n implies differ(a, b, n)
} by {
    if a != b and a.length <= n and b.length <= n {
        define p(la: List[T]) -> Bool {
            forall(lb: List[T]) {
                la != lb and la.length <= n and lb.length <= n implies differ(la, lb, n)
            }
        }

        // Base case: a = nil
        forall(lb: List[T]) {
            if List.nil[T] != lb and List.nil[T].length <= n and lb.length <= n {
                // If nil != lb, then lb = cons(hb, tb) for some hb, tb
                let (hb: T, tb: List[T]) satisfy {
                    lb = List.cons(hb, tb)
                }

                // At index 0: nil.get_idx(0) = none, cons.get_idx(0) = some(hb)
                lb.get_idx(0) = Option.some(hb)
                List.nil[T].get_idx(0) != lb.get_idx(0)

                // lb.length >= 1, and lb.length <= n, so n >= 1, so 0 < n

                // Therefore differ holds
                differ(List.nil[T], lb, n)
            }
        }
        not exists(lb: List[T]) {
            List.nil[T] != lb and List.nil[T].length <= n and lb.length <= n and not differ(List.nil[T], lb, n)
        }
        p(List.nil)

        // Inductive case: a = cons(ha, ta)
        forall(ha: T, ta: List[T]) {
            if p(ta) {
                forall(lb: List[T]) {
                    if List.cons(ha, ta) != lb and List.cons(ha, ta).length <= n and lb.length <= n {
                        if lb = List.nil[T] {
                            // cons != nil
                            List.cons(ha, ta).get_idx(0) = Option.some(ha)
                            List.cons(ha, ta).get_idx(0) != lb.get_idx(0)
                            not n < Nat.0
                            n < Nat.0 or Nat.0 < n or n = Nat.0
                            List.nil[T] != lb or lb.get_idx(Nat.0) = Option.none[T]
                            (List.cons(ha, ta).length <= n) = (n >= List.cons(ha, ta).length)
                            not n >= List.cons(ha, ta).length or List.cons(ha, ta).get_idx(n) = Option.none[T]

                            // Therefore differ holds
                            differ(List.cons(ha, ta), lb, n)
                        } else {
                            // lb = cons(hb, tb) for some hb, tb
                            let (hb: T, tb: List[T]) satisfy {
                                lb = List.cons(hb, tb)
                            }

                            // Show n >= 1 since both lists are non-empty

                            List.cons(ha, ta).get_idx(0) = Option.some(ha)
                            List.cons(hb, tb).get_idx(0) = Option.some(hb)

                            if ha = hb {
                                // Heads equal, so tails must differ

                                // Apply induction
                                tb.length < n

                                // From p(ta), we get differ(ta, tb, n)
                                differ(ta, tb, n)

                                // Expand differ to get the witness
                                let (i: Nat) satisfy {
                                    i < n and ta.get_idx(i) != tb.get_idx(i)
                                }

                                // Show i.suc < n
                                if i.suc >= n {
                                    let (ip: Nat) satisfy { ip.suc = n }
                                    i = ip
                                    ta.length <= ip
                                    tb.length <= ip
                                    (ta.length <= ip) = (ip >= ta.length)
                                    (tb.length <= ip) = (ip >= tb.length)
                                    not ip >= ta.length or ta.get_idx(ip) = Option.none[T]
                                    not ip >= tb.length or tb.get_idx(ip) = Option.none[T]
                                    false
                                }

                                // Use the fact that get_idx on cons with index > 0 accesses the tail
                                // i.suc = i + 1, and i + 1 > 0

                                // By definition/theorem: cons.get_idx(i+1) = tail.get_idx(i) when i+1 > 0
                                // Or equivalently: cons.get_idx(i.suc) = tail.get_idx((i.suc) - 1) when i.suc > 0
                                // And (i.suc) - 1 = i
                                i.suc > 0
                                List.cons(ha, ta).get_idx(i.suc) = ta.get_idx(i)

                                List.cons(hb, tb).get_idx(i.suc) = tb.get_idx(i)

                                // So the cons lists differ at i.suc < n
                                i.suc < n

                                // Explicate: get_idx values differ

                                // Therefore differ holds in this case
                                differ(List.cons(ha, ta), lb, n)
                            } else {
                                // Heads differ
                                0 < n

                                // Explicate: get_idx(0) values
                                List.cons(ha, ta).get_idx(0) = Option.some(ha)
                                lb.get_idx(0) = Option.some(hb)
                                ha != hb
                                List.cons(ha, ta).get_idx(0) != lb.get_idx(0)

                                // Therefore differ holds in this case
                                differ(List.cons(ha, ta), lb, n)
                            }

                            // In both branches we've shown differ holds
                            differ(List.cons(ha, ta), lb, n)
                        }

                        // The conclusion holds for this case
                    }
                }
                p(List.cons(ha, ta))
            }
        }

        p(a)
        // p(a) gives us differ(a, b, n), which expands to the existential
        differ(a, b, n)
    }
}

/// The partial sum of the zero index equals the identity element.
theorem partial_zero[A: AddCommMonoid](f: Nat -> A) {
    partial(f, Nat.0) = A.0
}


/// List extensionality: two lists with the same length that agree at all indices are equal.
theorem list_extensionality[T](a: List[T], b: List[T]) {
    a.length = b.length and (forall(i: Nat) { i < a.length implies a.get_idx(i) = b.get_idx(i) }) implies a = b
} by {
    if a.length = b.length and (forall(i: Nat) { i < a.length implies a.get_idx(i) = b.get_idx(i) }) {
        // Proof by contradiction
        if a != b {
            // Apply lists_differ_at_index with n = a.length
            differ(a, b, a.length)

            // This means there exists i < a.length where they differ

            // But we assumed they agree at all indices

            // Contradiction
        }

        // Therefore a = b
        a = b
    }
}
