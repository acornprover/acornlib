/// Finite meets and joins over non-empty lists.

from list.list_base import List
from lattice import MeetSemilattice, JoinSemilattice, meet_lte_left, meet_lte_right,
    lte_meet_of_bounds, lte_join_left, lte_join_right, join_lte_of_bounds,
    lte_meet_iff, join_lte_iff, meet_assoc_rev, join_assoc_rev
from order import PartialOrder, lte_refl, lte_trans

/// The meet of a non-empty list.
define list_meet[S: MeetSemilattice](head: S, tail: List[S]) -> S {
    match tail {
        List.nil {
            head
        }
        List.cons(next, rest) {
            head.meet(list_meet(next, rest))
        }
    }
}

/// The join of a non-empty list.
define list_join[S: JoinSemilattice](head: S, tail: List[S]) -> S {
    match tail {
        List.nil {
            head
        }
        List.cons(next, rest) {
            head.join(list_join(next, rest))
        }
    }
}

/// The infimum of a non-empty list.
define list_inf[S: MeetSemilattice](head: S, tail: List[S]) -> S {
    list_meet(head, tail)
}

/// The supremum of a non-empty list.
define list_sup[S: JoinSemilattice](head: S, tail: List[S]) -> S {
    list_join(head, tail)
}

/// True if a point is below every element of a list.
define list_lower_bound[S: PartialOrder](items: List[S], lower: S) -> Bool {
    forall(x: S) {
        items.contains(x) implies lower <= x
    }
}

/// True if a point is above every element of a list.
define list_upper_bound[S: PartialOrder](items: List[S], upper: S) -> Bool {
    forall(x: S) {
        items.contains(x) implies x <= upper
    }
}

/// A lower bound for a cons list is below the head.
theorem list_lower_bound_head[S: PartialOrder](head: S, tail: List[S], lower: S) {
    list_lower_bound(List.cons(head, tail), lower) implies lower <= head
} by {
    if list_lower_bound(List.cons(head, tail), lower) {
        list_lower_bound(List.cons(head, tail), lower) = forall(x: S) {
            List.cons(head, tail).contains(x) implies lower <= x
        }
        lower <= head
    }
}

/// A lower bound for a cons list is a lower bound for the tail.
theorem list_lower_bound_tail[S: PartialOrder](head: S, tail: List[S], lower: S) {
    list_lower_bound(List.cons(head, tail), lower) implies list_lower_bound(tail, lower)
} by {
    if list_lower_bound(List.cons(head, tail), lower) {
        list_lower_bound(List.cons(head, tail), lower) = forall(x: S) {
            List.cons(head, tail).contains(x) implies lower <= x
        }
        forall(x: S) {
            if tail.contains(x) {
                List.cons(head, tail).contains(x)
                lower <= x
            }
        }
        list_lower_bound(tail, lower)
    }
}

/// An upper bound for a cons list is above the head.
theorem list_upper_bound_head[S: PartialOrder](head: S, tail: List[S], upper: S) {
    list_upper_bound(List.cons(head, tail), upper) implies head <= upper
} by {
    if list_upper_bound(List.cons(head, tail), upper) {
        list_upper_bound(List.cons(head, tail), upper) = forall(x: S) {
            List.cons(head, tail).contains(x) implies x <= upper
        }
        head <= upper
    }
}

/// An upper bound for a cons list is an upper bound for the tail.
theorem list_upper_bound_tail[S: PartialOrder](head: S, tail: List[S], upper: S) {
    list_upper_bound(List.cons(head, tail), upper) implies list_upper_bound(tail, upper)
} by {
    if list_upper_bound(List.cons(head, tail), upper) {
        list_upper_bound(List.cons(head, tail), upper) = forall(x: S) {
            List.cons(head, tail).contains(x) implies x <= upper
        }
        forall(x: S) {
            if tail.contains(x) {
                List.cons(head, tail).contains(x)
                x <= upper
            }
        }
        list_upper_bound(tail, upper)
    }
}

/// Lower bounds for a cons list are exactly lower bounds for the head and tail.
theorem list_lower_bound_cons_iff[S: PartialOrder](head: S, tail: List[S], lower: S) {
    list_lower_bound(List.cons(head, tail), lower) =
    (lower <= head and list_lower_bound(tail, lower))
} by {
    if list_lower_bound(List.cons(head, tail), lower) {
        list_lower_bound_head(head, tail, lower)
        list_lower_bound_tail(head, tail, lower)
        list_lower_bound(tail, lower)
        lower <= head and list_lower_bound(tail, lower)
    }
    if lower <= head and list_lower_bound(tail, lower) {
        forall(x: S) {
            if List.cons(head, tail).contains(x) {
                if head = x {
                    lower <= x
                } else {
                    list_lower_bound(tail, lower) = forall(y: S) {
                        tail.contains(y) implies lower <= y
                    }
                    lower <= x
                }
            }
        }
        list_lower_bound(List.cons(head, tail), lower)
    }
}

/// Upper bounds for a cons list are exactly upper bounds for the head and tail.
theorem list_upper_bound_cons_iff[S: PartialOrder](head: S, tail: List[S], upper: S) {
    list_upper_bound(List.cons(head, tail), upper) =
    (head <= upper and list_upper_bound(tail, upper))
} by {
    if list_upper_bound(List.cons(head, tail), upper) {
        list_upper_bound_head(head, tail, upper)
        list_upper_bound_tail(head, tail, upper)
        list_upper_bound(tail, upper)
        head <= upper and list_upper_bound(tail, upper)
    }
    if head <= upper and list_upper_bound(tail, upper) {
        forall(x: S) {
            if List.cons(head, tail).contains(x) {
                if head = x {
                    x <= upper
                } else {
                    list_upper_bound(tail, upper) = forall(y: S) {
                        tail.contains(y) implies y <= upper
                    }
                    x <= upper
                }
            }
        }
        list_upper_bound(List.cons(head, tail), upper)
    }
}

/// A lower bound for a concatenation is a lower bound for the left list.
theorem list_lower_bound_add_left[S: PartialOrder](left: List[S], right: List[S], lower: S) {
    list_lower_bound(left + right, lower) implies list_lower_bound(left, lower)
} by {
    define p(l: List[S]) -> Bool {
        list_lower_bound(l + right, lower) implies list_lower_bound(l, lower)
    }
    list_lower_bound(List.nil[S], lower)
    p(List.nil[S])
    forall(head: S, tail: List[S]) {
        if p(tail) {
            if list_lower_bound(List.cons(head, tail) + right, lower) {
                list_lower_bound(List.cons(head, tail + right), lower)
                list_lower_bound_cons_iff(head, tail + right, lower)
                list_lower_bound_cons_iff(head, tail, lower)
                list_lower_bound(List.cons(head, tail), lower)
            }
            p(List.cons(head, tail))
        }
    }
    p(left)
}

/// A lower bound for a concatenation is a lower bound for the right list.
theorem list_lower_bound_add_right[S: PartialOrder](left: List[S], right: List[S], lower: S) {
    list_lower_bound(left + right, lower) implies list_lower_bound(right, lower)
} by {
    define p(l: List[S]) -> Bool {
        list_lower_bound(l + right, lower) implies list_lower_bound(right, lower)
    }
    p(List.nil[S])
    forall(head: S, tail: List[S]) {
        if p(tail) {
            if list_lower_bound(List.cons(head, tail) + right, lower) {
                list_lower_bound_tail(head, tail + right, lower)
                list_lower_bound(tail + right, lower)
                list_lower_bound(right, lower)
            }
            p(List.cons(head, tail))
        }
    }
    p(left)
}

/// Lower bounds for a concatenation are exactly lower bounds for both lists.
theorem list_lower_bound_add_iff[S: PartialOrder](left: List[S], right: List[S], lower: S) {
    list_lower_bound(left + right, lower) =
    (list_lower_bound(left, lower) and list_lower_bound(right, lower))
} by {
    if list_lower_bound(left + right, lower) {
        list_lower_bound_add_left(left, right, lower)
        list_lower_bound_add_right(left, right, lower)
        list_lower_bound(right, lower)
        list_lower_bound(left, lower) and list_lower_bound(right, lower)
    }
    if list_lower_bound(left, lower) and list_lower_bound(right, lower) {
        forall(x: S) {
            if (left + right).contains(x) {
                if left.contains(x) {
                    list_lower_bound(left, lower) = forall(y: S) {
                        left.contains(y) implies lower <= y
                    }
                    lower <= x
                } else {
                    right.contains(x)
                    list_lower_bound(right, lower) = forall(y: S) {
                        right.contains(y) implies lower <= y
                    }
                    lower <= x
                }
            }
        }
        list_lower_bound(left + right, lower)
    }
}

/// An upper bound for a concatenation is an upper bound for the left list.
theorem list_upper_bound_add_left[S: PartialOrder](left: List[S], right: List[S], upper: S) {
    list_upper_bound(left + right, upper) implies list_upper_bound(left, upper)
} by {
    define p(l: List[S]) -> Bool {
        list_upper_bound(l + right, upper) implies list_upper_bound(l, upper)
    }
    list_upper_bound(List.nil[S], upper)
    p(List.nil[S])
    forall(head: S, tail: List[S]) {
        if p(tail) {
            if list_upper_bound(List.cons(head, tail) + right, upper) {
                list_upper_bound(List.cons(head, tail + right), upper)
                list_upper_bound_cons_iff(head, tail + right, upper)
                list_upper_bound_cons_iff(head, tail, upper)
                list_upper_bound(List.cons(head, tail), upper)
            }
            p(List.cons(head, tail))
        }
    }
    p(left)
}

/// An upper bound for a concatenation is an upper bound for the right list.
theorem list_upper_bound_add_right[S: PartialOrder](left: List[S], right: List[S], upper: S) {
    list_upper_bound(left + right, upper) implies list_upper_bound(right, upper)
} by {
    define p(l: List[S]) -> Bool {
        list_upper_bound(l + right, upper) implies list_upper_bound(right, upper)
    }
    p(List.nil[S])
    forall(head: S, tail: List[S]) {
        if p(tail) {
            if list_upper_bound(List.cons(head, tail) + right, upper) {
                list_upper_bound_tail(head, tail + right, upper)
                list_upper_bound(tail + right, upper)
                list_upper_bound(right, upper)
            }
            p(List.cons(head, tail))
        }
    }
    p(left)
}

/// Upper bounds for a concatenation are exactly upper bounds for both lists.
theorem list_upper_bound_add_iff[S: PartialOrder](left: List[S], right: List[S], upper: S) {
    list_upper_bound(left + right, upper) =
    (list_upper_bound(left, upper) and list_upper_bound(right, upper))
} by {
    if list_upper_bound(left + right, upper) {
        list_upper_bound_add_left(left, right, upper)
        list_upper_bound_add_right(left, right, upper)
        list_upper_bound(right, upper)
        list_upper_bound(left, upper) and list_upper_bound(right, upper)
    }
    if list_upper_bound(left, upper) and list_upper_bound(right, upper) {
        forall(x: S) {
            if (left + right).contains(x) {
                if left.contains(x) {
                    list_upper_bound(left, upper) = forall(y: S) {
                        left.contains(y) implies y <= upper
                    }
                    x <= upper
                } else {
                    right.contains(x)
                    list_upper_bound(right, upper) = forall(y: S) {
                        right.contains(y) implies y <= upper
                    }
                    x <= upper
                }
            }
        }
        list_upper_bound(left + right, upper)
    }
}

/// Every point is a lower bound for the empty list.
theorem list_lower_bound_nil[S: PartialOrder](lower: S) {
    list_lower_bound(List.nil[S], lower)
} by {
    forall(x: S) {
        if List.nil[S].contains(x) {
            false
        }
    }
}

/// Every point is an upper bound for the empty list.
theorem list_upper_bound_nil[S: PartialOrder](upper: S) {
    list_upper_bound(List.nil[S], upper)
} by {
    forall(x: S) {
        if List.nil[S].contains(x) {
            false
        }
    }
}

/// A lower bound for a singleton list is exactly a point below the singleton element.
theorem list_lower_bound_singleton_iff[S: PartialOrder](item: S, lower: S) {
    list_lower_bound(List.singleton(item), lower) = (lower <= item)
} by {
    list_lower_bound_cons_iff(item, List.nil[S], lower)
    list_lower_bound_nil(lower)
}

/// An upper bound for a singleton list is exactly a point above the singleton element.
theorem list_upper_bound_singleton_iff[S: PartialOrder](item: S, upper: S) {
    list_upper_bound(List.singleton(item), upper) = (item <= upper)
} by {
    list_upper_bound_cons_iff(item, List.nil[S], upper)
    list_upper_bound_nil(upper)
}

/// Lower bounds after append are exactly lower bounds for the old list and the appended element.
theorem list_lower_bound_append_iff[S: PartialOrder](items: List[S], last: S, lower: S) {
    list_lower_bound(items.append(last), lower) =
    (list_lower_bound(items, lower) and lower <= last)
} by {
    list_lower_bound_add_iff(items, List.singleton(last), lower)
    list_lower_bound_singleton_iff(last, lower)
}

/// Upper bounds after append are exactly upper bounds for the old list and the appended element.
theorem list_upper_bound_append_iff[S: PartialOrder](items: List[S], last: S, upper: S) {
    list_upper_bound(items.append(last), upper) =
    (list_upper_bound(items, upper) and last <= upper)
} by {
    list_upper_bound_add_iff(items, List.singleton(last), upper)
    list_upper_bound_singleton_iff(last, upper)
}

/// Lower bounds are downward closed.
theorem list_lower_bound_monotone[S: PartialOrder](items: List[S], lower: S, smaller: S) {
    list_lower_bound(items, lower) and smaller <= lower implies list_lower_bound(items, smaller)
} by {
    if list_lower_bound(items, lower) and smaller <= lower {
        forall(x: S) {
            if items.contains(x) {
                list_lower_bound(items, lower) = forall(y: S) {
                    items.contains(y) implies lower <= y
                }
                lte_trans(smaller, lower, x)
                smaller <= x
            }
        }
        list_lower_bound(items, smaller)
    }
}

/// Upper bounds are upward closed.
theorem list_upper_bound_monotone[S: PartialOrder](items: List[S], upper: S, larger: S) {
    list_upper_bound(items, upper) and upper <= larger implies list_upper_bound(items, larger)
} by {
    if list_upper_bound(items, upper) and upper <= larger {
        forall(x: S) {
            if items.contains(x) {
                list_upper_bound(items, upper) = forall(y: S) {
                    items.contains(y) implies y <= upper
                }
                lte_trans(x, upper, larger)
                x <= larger
            }
        }
        list_upper_bound(items, larger)
    }
}

/// The meet of a non-empty list is below its head.
theorem list_meet_lte_head[S: MeetSemilattice](head: S, tail: List[S]) {
    list_meet(head, tail) <= head
} by {
    define p(t: List[S]) -> Bool {
        forall(h: S) {
            list_meet(h, t) <= h
        }
    }
    forall(h: S) {
        lte_refl(h)
        list_meet(h, List.nil[S]) <= h
    }
    p(List.nil[S])
    forall(next: S, rest: List[S]) {
        if p(rest) {
            forall(h: S) {
                meet_lte_left(h, list_meet(next, rest))
                list_meet(h, List.cons(next, rest)) <= h
            }
            p(List.cons(next, rest))
        }
    }
    p(tail)
}

/// The meet of a cons list is below the meet of its non-empty tail.
theorem list_meet_lte_tail_meet[S: MeetSemilattice](head: S, next: S, rest: List[S]) {
    list_meet(head, List.cons(next, rest)) <= list_meet(next, rest)
} by {
    meet_lte_right(head, list_meet(next, rest))
}

/// The meet of a non-empty list is below every element of the list.
theorem list_meet_lte_contains[S: MeetSemilattice](head: S, tail: List[S], x: S) {
    List.cons(head, tail).contains(x) implies list_meet(head, tail) <= x
} by {
    define p(t: List[S]) -> Bool {
        forall(h: S, y: S) {
            List.cons(h, t).contains(y) implies list_meet(h, t) <= y
        }
    }
    forall(h: S, y: S) {
        if List.cons(h, List.nil[S]).contains(y) {
            if h = y {
                lte_refl(h)
                list_meet(h, List.nil[S]) <= y
            } else {
                List.nil[S].contains(y)
                false
            }
        }
    }
    p(List.nil[S])
    forall(next: S, rest: List[S]) {
        if p(rest) {
            forall(h: S, y: S) {
                if List.cons(h, List.cons(next, rest)).contains(y) {
                    if h = y {
                        meet_lte_left(h, list_meet(next, rest))
                        list_meet(h, List.cons(next, rest)) <= y
                    } else {
                        List.cons(next, rest).contains(y)
                        meet_lte_right(h, list_meet(next, rest))
                        list_meet(next, rest) <= y
                        lte_trans(list_meet(h, List.cons(next, rest)), list_meet(next, rest), y)
                        list_meet(h, List.cons(next, rest)) <= y
                    }
                }
            }
            p(List.cons(next, rest))
        }
    }
    p(tail)
}

/// The head is below the join of a non-empty list.
theorem head_lte_list_join[S: JoinSemilattice](head: S, tail: List[S]) {
    head <= list_join(head, tail)
} by {
    define p(t: List[S]) -> Bool {
        forall(h: S) {
            h <= list_join(h, t)
        }
    }
    forall(h: S) {
        lte_refl(h)
        h <= list_join(h, List.nil[S])
    }
    p(List.nil[S])
    forall(next: S, rest: List[S]) {
        if p(rest) {
            forall(h: S) {
                lte_join_left(h, list_join(next, rest))
                h <= list_join(h, List.cons(next, rest))
            }
            p(List.cons(next, rest))
        }
    }
    p(tail)
}

/// The join of a non-empty tail is below the join of the whole cons list.
theorem tail_join_lte_list_join[S: JoinSemilattice](head: S, next: S, rest: List[S]) {
    list_join(next, rest) <= list_join(head, List.cons(next, rest))
} by {
    lte_join_right(head, list_join(next, rest))
}

/// Every element of a non-empty list is below its join.
theorem contains_lte_list_join[S: JoinSemilattice](head: S, tail: List[S], x: S) {
    List.cons(head, tail).contains(x) implies x <= list_join(head, tail)
} by {
    define p(t: List[S]) -> Bool {
        forall(h: S, y: S) {
            List.cons(h, t).contains(y) implies y <= list_join(h, t)
        }
    }
    forall(h: S, y: S) {
        if List.cons(h, List.nil[S]).contains(y) {
            if h = y {
                lte_refl(h)
                y <= list_join(h, List.nil[S])
            } else {
                List.nil[S].contains(y)
                false
            }
        }
    }
    p(List.nil[S])
    forall(next: S, rest: List[S]) {
        if p(rest) {
            forall(h: S, y: S) {
                if List.cons(h, List.cons(next, rest)).contains(y) {
                    if h = y {
                        lte_join_left(h, list_join(next, rest))
                        y <= list_join(h, List.cons(next, rest))
                    } else {
                        List.cons(next, rest).contains(y)
                        lte_join_right(h, list_join(next, rest))
                        lte_trans(y, list_join(next, rest), list_join(h, List.cons(next, rest)))
                        y <= list_join(h, List.cons(next, rest))
                    }
                }
            }
            p(List.cons(next, rest))
        }
    }
    p(tail)
}

/// Any lower bound for every element of a non-empty list is below its meet.
theorem lte_list_meet_of_contains_bounds[S: MeetSemilattice](c: S, head: S, tail: List[S]) {
    list_lower_bound(List.cons(head, tail), c) implies
    c <= list_meet(head, tail)
} by {
    define p(t: List[S]) -> Bool {
        forall(h: S, lower: S) {
            list_lower_bound(List.cons(h, t), lower) implies
            lower <= list_meet(h, t)
        }
    }
    forall(h: S, lower: S) {
        if list_lower_bound(List.cons(h, List.nil[S]), lower) {
            list_lower_bound_head(h, List.nil[S], lower)
            lower <= list_meet(h, List.nil[S])
        }
    }
    p(List.nil[S])
    forall(next: S, rest: List[S]) {
        if p(rest) {
            forall(h: S, lower: S) {
                if list_lower_bound(List.cons(h, List.cons(next, rest)), lower) {
                    list_lower_bound_head(h, List.cons(next, rest), lower)
                    lower <= h
                    list_lower_bound_tail(h, List.cons(next, rest), lower)
                    lower <= list_meet(next, rest)
                    lte_meet_of_bounds(lower, h, list_meet(next, rest))
                    lower <= list_meet(h, List.cons(next, rest))
                }
            }
            p(List.cons(next, rest))
        }
    }
    p(tail)
}

/// The meet of a non-empty list is the greatest lower bound of its elements.
theorem lte_list_meet_iff_contains_bounds[S: MeetSemilattice](c: S, head: S, tail: List[S]) {
    c <= list_meet(head, tail) = list_lower_bound(List.cons(head, tail), c)
} by {
    if c <= list_meet(head, tail) {
        forall(x: S) {
            if List.cons(head, tail).contains(x) {
                list_meet_lte_contains(head, tail, x)
                lte_trans(c, list_meet(head, tail), x)
                c <= x
            }
        }
        list_lower_bound(List.cons(head, tail), c)
    }
    if list_lower_bound(List.cons(head, tail), c) {
        lte_list_meet_of_contains_bounds(c, head, tail)
        c <= list_meet(head, tail)
    }
}

/// The meet of a non-empty list is a lower bound for that list.
theorem list_meet_is_lower_bound[S: MeetSemilattice](head: S, tail: List[S]) {
    list_lower_bound(List.cons(head, tail), list_meet(head, tail))
} by {
    forall(x: S) {
        if List.cons(head, tail).contains(x) {
            list_meet_lte_contains(head, tail, x)
            list_meet(head, tail) <= x
        }
    }
}

/// A point is below the list meet exactly when it is a lower bound for the list.
theorem le_list_meet_iff[S: MeetSemilattice](c: S, head: S, tail: List[S]) {
    c <= list_meet(head, tail) = list_lower_bound(List.cons(head, tail), c)
} by {
    lte_list_meet_iff_contains_bounds(c, head, tail)
}

/// Any upper bound for every element of a non-empty list is above its join.
theorem list_join_lte_of_contains_bounds[S: JoinSemilattice](head: S, tail: List[S], c: S) {
    list_upper_bound(List.cons(head, tail), c) implies
    list_join(head, tail) <= c
} by {
    define p(t: List[S]) -> Bool {
        forall(h: S, upper: S) {
            list_upper_bound(List.cons(h, t), upper) implies
            list_join(h, t) <= upper
        }
    }
    forall(h: S, upper: S) {
        if list_upper_bound(List.cons(h, List.nil[S]), upper) {
            list_upper_bound_head(h, List.nil[S], upper)
            list_join(h, List.nil[S]) <= upper
        }
    }
    p(List.nil[S])
    forall(next: S, rest: List[S]) {
        if p(rest) {
            forall(h: S, upper: S) {
                if list_upper_bound(List.cons(h, List.cons(next, rest)), upper) {
                    list_upper_bound_head(h, List.cons(next, rest), upper)
                    h <= upper
                    list_upper_bound_tail(h, List.cons(next, rest), upper)
                    list_join(next, rest) <= upper
                    join_lte_of_bounds(h, list_join(next, rest), upper)
                    list_join(h, List.cons(next, rest)) <= upper
                }
            }
            p(List.cons(next, rest))
        }
    }
    p(tail)
}

/// The join of a non-empty list is the least upper bound of its elements.
theorem list_join_lte_iff_contains_bounds[S: JoinSemilattice](head: S, tail: List[S], c: S) {
    list_join(head, tail) <= c = list_upper_bound(List.cons(head, tail), c)
} by {
    if list_join(head, tail) <= c {
        forall(x: S) {
            if List.cons(head, tail).contains(x) {
                contains_lte_list_join(head, tail, x)
                lte_trans(x, list_join(head, tail), c)
                x <= c
            }
        }
        list_upper_bound(List.cons(head, tail), c)
    }
    if list_upper_bound(List.cons(head, tail), c) {
        list_join_lte_of_contains_bounds(head, tail, c)
        list_join(head, tail) <= c
    }
}

/// The join of a non-empty list is an upper bound for that list.
theorem list_join_is_upper_bound[S: JoinSemilattice](head: S, tail: List[S]) {
    list_upper_bound(List.cons(head, tail), list_join(head, tail))
} by {
    forall(x: S) {
        if List.cons(head, tail).contains(x) {
            contains_lte_list_join(head, tail, x)
            x <= list_join(head, tail)
        }
    }
}

/// The list join is below a point exactly when the point is an upper bound for the list.
theorem list_join_le_iff[S: JoinSemilattice](head: S, tail: List[S], c: S) {
    list_join(head, tail) <= c = list_upper_bound(List.cons(head, tail), c)
} by {
    list_join_lte_iff_contains_bounds(head, tail, c)
}

/// The meet over a concatenation with a non-empty right list is the meet of the two list meets.
theorem list_meet_add_cons[S: MeetSemilattice](head: S, tail: List[S], next: S, rest: List[S]) {
    list_meet(head, tail + List.cons(next, rest)) =
    list_meet(head, tail).meet(list_meet(next, rest))
} by {
    define p(t: List[S]) -> Bool {
        forall(h: S, n: S, r: List[S]) {
            list_meet(h, t + List.cons(n, r)) =
            list_meet(h, t).meet(list_meet(n, r))
        }
    }
    forall(h: S, n: S, r: List[S]) {
        list_meet(h, List.nil[S] + List.cons(n, r)) =
        list_meet(h, List.nil[S]).meet(list_meet(n, r))
    }
    p(List.nil[S])
    forall(next_tail: S, rest_tail: List[S]) {
        if p(rest_tail) {
            forall(h: S, n: S, r: List[S]) {
                let right_meet = list_meet(n, r)
                let tail_meet = list_meet(next_tail, rest_tail)
                list_meet(h, List.cons(next_tail, rest_tail) + List.cons(n, r)) =
                h.meet(list_meet(next_tail, rest_tail + List.cons(n, r)))
                list_meet(next_tail, rest_tail + List.cons(n, r)) =
                list_meet(next_tail, rest_tail).meet(list_meet(n, r))
                meet_assoc_rev(h, tail_meet, right_meet)
                list_meet(h, List.cons(next_tail, rest_tail) + List.cons(n, r)) =
                list_meet(h, List.cons(next_tail, rest_tail)).meet(list_meet(n, r))
            }
            p(List.cons(next_tail, rest_tail))
        }
    }
    p(tail)
}

/// The join over a concatenation with a non-empty right list is the join of the two list joins.
theorem list_join_add_cons[S: JoinSemilattice](head: S, tail: List[S], next: S, rest: List[S]) {
    list_join(head, tail + List.cons(next, rest)) =
    list_join(head, tail).join(list_join(next, rest))
} by {
    define p(t: List[S]) -> Bool {
        forall(h: S, n: S, r: List[S]) {
            list_join(h, t + List.cons(n, r)) =
            list_join(h, t).join(list_join(n, r))
        }
    }
    forall(h: S, n: S, r: List[S]) {
        list_join(h, List.nil[S] + List.cons(n, r)) =
        list_join(h, List.nil[S]).join(list_join(n, r))
    }
    p(List.nil[S])
    forall(next_tail: S, rest_tail: List[S]) {
        if p(rest_tail) {
            forall(h: S, n: S, r: List[S]) {
                let right_join = list_join(n, r)
                let tail_join = list_join(next_tail, rest_tail)
                list_join(h, List.cons(next_tail, rest_tail) + List.cons(n, r)) =
                h.join(list_join(next_tail, rest_tail + List.cons(n, r)))
                list_join(next_tail, rest_tail + List.cons(n, r)) =
                list_join(next_tail, rest_tail).join(list_join(n, r))
                join_assoc_rev(h, tail_join, right_join)
                list_join(h, List.cons(next_tail, rest_tail) + List.cons(n, r)) =
                list_join(h, List.cons(next_tail, rest_tail)).join(list_join(n, r))
            }
            p(List.cons(next_tail, rest_tail))
        }
    }
    p(tail)
}

/// The meet over a concatenation is below the meet of the left non-empty list.
theorem list_meet_add_cons_lte_left[S: MeetSemilattice](head: S, tail: List[S], next: S, rest: List[S]) {
    list_meet(head, tail + List.cons(next, rest)) <= list_meet(head, tail)
} by {
    list_meet_add_cons(head, tail, next, rest)
    meet_lte_left(list_meet(head, tail), list_meet(next, rest))
}

/// The meet over a concatenation is below the meet of the right non-empty list.
theorem list_meet_add_cons_lte_right[S: MeetSemilattice](head: S, tail: List[S], next: S, rest: List[S]) {
    list_meet(head, tail + List.cons(next, rest)) <= list_meet(next, rest)
} by {
    list_meet_add_cons(head, tail, next, rest)
    meet_lte_right(list_meet(head, tail), list_meet(next, rest))
}

/// The join of the left non-empty list is below the join over a concatenation.
theorem list_join_lte_add_cons_left[S: JoinSemilattice](head: S, tail: List[S], next: S, rest: List[S]) {
    list_join(head, tail) <= list_join(head, tail + List.cons(next, rest))
} by {
    list_join_add_cons(head, tail, next, rest)
    lte_join_left(list_join(head, tail), list_join(next, rest))
}

/// The join of the right non-empty list is below the join over a concatenation.
theorem list_join_lte_add_cons_right[S: JoinSemilattice](head: S, tail: List[S], next: S, rest: List[S]) {
    list_join(next, rest) <= list_join(head, tail + List.cons(next, rest))
} by {
    list_join_add_cons(head, tail, next, rest)
    lte_join_right(list_join(head, tail), list_join(next, rest))
}

/// A point is below the meet over a concatenation exactly when it is below both list meets.
theorem le_list_meet_add_cons_iff[S: MeetSemilattice](c: S, head: S, tail: List[S],
        next: S, rest: List[S]) {
    c <= list_meet(head, tail + List.cons(next, rest)) =
    (c <= list_meet(head, tail) and c <= list_meet(next, rest))
} by {
    list_meet_add_cons(head, tail, next, rest)
    lte_meet_iff(c, list_meet(head, tail), list_meet(next, rest))
}

/// The join over a concatenation is below a point exactly when both list joins are below it.
theorem list_join_add_cons_le_iff[S: JoinSemilattice](head: S, tail: List[S],
        next: S, rest: List[S], c: S) {
    list_join(head, tail + List.cons(next, rest)) <= c =
    (list_join(head, tail) <= c and list_join(next, rest) <= c)
} by {
    list_join_add_cons(head, tail, next, rest)
    join_lte_iff(list_join(head, tail), list_join(next, rest), c)
}

/// Appending one element meets the old list meet with that element.
theorem list_meet_append[S: MeetSemilattice](head: S, tail: List[S], last: S) {
    list_meet(head, tail.append(last)) = list_meet(head, tail).meet(last)
} by {
    List.singleton(last) = List.cons(last, List.nil[S])
    list_meet_add_cons(head, tail, last, List.nil[S])
}

/// Appending one element joins the old list join with that element.
theorem list_join_append[S: JoinSemilattice](head: S, tail: List[S], last: S) {
    list_join(head, tail.append(last)) = list_join(head, tail).join(last)
} by {
    List.singleton(last) = List.cons(last, List.nil[S])
    list_join_add_cons(head, tail, last, List.nil[S])
}

/// The meet after appending an element is below the previous list meet.
theorem list_meet_append_lte_list_meet[S: MeetSemilattice](head: S, tail: List[S], last: S) {
    list_meet(head, tail.append(last)) <= list_meet(head, tail)
} by {
    list_meet_append(head, tail, last)
    meet_lte_left(list_meet(head, tail), last)
}

/// The meet after appending an element is below that element.
theorem list_meet_append_lte_last[S: MeetSemilattice](head: S, tail: List[S], last: S) {
    list_meet(head, tail.append(last)) <= last
} by {
    list_meet_append(head, tail, last)
    meet_lte_right(list_meet(head, tail), last)
}

/// The previous list join is below the join after appending an element.
theorem list_join_lte_append_join[S: JoinSemilattice](head: S, tail: List[S], last: S) {
    list_join(head, tail) <= list_join(head, tail.append(last))
} by {
    list_join_append(head, tail, last)
    lte_join_left(list_join(head, tail), last)
}

/// The appended element is below the join after appending it.
theorem last_lte_list_join_append[S: JoinSemilattice](head: S, tail: List[S], last: S) {
    last <= list_join(head, tail.append(last))
} by {
    list_join_append(head, tail, last)
    lte_join_right(list_join(head, tail), last)
}

/// A point is below the meet after append exactly when it is below the previous meet and the appended element.
theorem le_list_meet_append_iff[S: MeetSemilattice](c: S, head: S, tail: List[S], last: S) {
    c <= list_meet(head, tail.append(last)) =
    (c <= list_meet(head, tail) and c <= last)
} by {
    list_meet_append(head, tail, last)
    lte_meet_iff(c, list_meet(head, tail), last)
}

/// The join after append is below a point exactly when the previous join and the appended element are below it.
theorem list_join_append_le_iff[S: JoinSemilattice](head: S, tail: List[S], last: S, c: S) {
    list_join(head, tail.append(last)) <= c =
    (list_join(head, tail) <= c and last <= c)
} by {
    list_join_append(head, tail, last)
    join_lte_iff(list_join(head, tail), last, c)
}

/// List infimum is the list meet.
theorem list_inf_eq_list_meet[S: MeetSemilattice](head: S, tail: List[S]) {
    list_inf(head, tail) = list_meet(head, tail)
}

/// List supremum is the list join.
theorem list_sup_eq_list_join[S: JoinSemilattice](head: S, tail: List[S]) {
    list_sup(head, tail) = list_join(head, tail)
}

/// The infimum of a non-empty list is below its head.
theorem list_inf_lte_head[S: MeetSemilattice](head: S, tail: List[S]) {
    list_inf(head, tail) <= head
} by {
    list_inf_eq_list_meet(head, tail)
    list_meet_lte_head(head, tail)
}

/// The infimum of a cons list is below the infimum of its non-empty tail.
theorem list_inf_lte_tail_inf[S: MeetSemilattice](head: S, next: S, rest: List[S]) {
    list_inf(head, List.cons(next, rest)) <= list_inf(next, rest)
} by {
    list_inf_eq_list_meet(head, List.cons(next, rest))
    list_inf_eq_list_meet(next, rest)
    list_meet_lte_tail_meet(head, next, rest)
}

/// The infimum of a non-empty list is below every element of the list.
theorem list_inf_lte_contains[S: MeetSemilattice](head: S, tail: List[S], x: S) {
    List.cons(head, tail).contains(x) implies list_inf(head, tail) <= x
} by {
    if List.cons(head, tail).contains(x) {
        list_inf_eq_list_meet(head, tail)
        list_meet_lte_contains(head, tail, x)
        list_inf(head, tail) <= x
    }
}

/// The head is below the supremum of a non-empty list.
theorem head_lte_list_sup[S: JoinSemilattice](head: S, tail: List[S]) {
    head <= list_sup(head, tail)
} by {
    list_sup_eq_list_join(head, tail)
    head_lte_list_join(head, tail)
}

/// The supremum of a non-empty tail is below the supremum of the whole cons list.
theorem tail_sup_lte_list_sup[S: JoinSemilattice](head: S, next: S, rest: List[S]) {
    list_sup(next, rest) <= list_sup(head, List.cons(next, rest))
} by {
    list_sup_eq_list_join(next, rest)
    list_sup_eq_list_join(head, List.cons(next, rest))
    tail_join_lte_list_join(head, next, rest)
}

/// Every element of a non-empty list is below its supremum.
theorem contains_lte_list_sup[S: JoinSemilattice](head: S, tail: List[S], x: S) {
    List.cons(head, tail).contains(x) implies x <= list_sup(head, tail)
} by {
    if List.cons(head, tail).contains(x) {
        list_sup_eq_list_join(head, tail)
        contains_lte_list_join(head, tail, x)
        x <= list_sup(head, tail)
    }
}

/// Any lower bound for every element of a non-empty list is below its infimum.
theorem lte_list_inf_of_contains_bounds[S: MeetSemilattice](c: S, head: S, tail: List[S]) {
    list_lower_bound(List.cons(head, tail), c) implies
    c <= list_inf(head, tail)
} by {
    if list_lower_bound(List.cons(head, tail), c) {
        list_inf_eq_list_meet(head, tail)
        lte_list_meet_of_contains_bounds(c, head, tail)
        c <= list_inf(head, tail)
    }
}

/// The infimum of a non-empty list is the greatest lower bound of its elements.
theorem lte_list_inf_iff_contains_bounds[S: MeetSemilattice](c: S, head: S, tail: List[S]) {
    c <= list_inf(head, tail) = list_lower_bound(List.cons(head, tail), c)
} by {
    list_inf_eq_list_meet(head, tail)
    lte_list_meet_iff_contains_bounds(c, head, tail)
}

/// The infimum of a non-empty list is a lower bound for that list.
theorem list_inf_is_lower_bound[S: MeetSemilattice](head: S, tail: List[S]) {
    list_lower_bound(List.cons(head, tail), list_inf(head, tail))
} by {
    list_inf_eq_list_meet(head, tail)
    list_meet_is_lower_bound(head, tail)
}

/// A point is below the list infimum exactly when it is a lower bound for the list.
theorem le_list_inf_iff[S: MeetSemilattice](c: S, head: S, tail: List[S]) {
    c <= list_inf(head, tail) = list_lower_bound(List.cons(head, tail), c)
} by {
    lte_list_inf_iff_contains_bounds(c, head, tail)
}

/// Any upper bound for every element of a non-empty list is above its supremum.
theorem list_sup_lte_of_contains_bounds[S: JoinSemilattice](head: S, tail: List[S], c: S) {
    list_upper_bound(List.cons(head, tail), c) implies
    list_sup(head, tail) <= c
} by {
    if list_upper_bound(List.cons(head, tail), c) {
        list_sup_eq_list_join(head, tail)
        list_join_lte_of_contains_bounds(head, tail, c)
        list_sup(head, tail) <= c
    }
}

/// The supremum of a non-empty list is the least upper bound of its elements.
theorem list_sup_lte_iff_contains_bounds[S: JoinSemilattice](head: S, tail: List[S], c: S) {
    list_sup(head, tail) <= c = list_upper_bound(List.cons(head, tail), c)
} by {
    list_sup_eq_list_join(head, tail)
    list_join_lte_iff_contains_bounds(head, tail, c)
}

/// The supremum of a non-empty list is an upper bound for that list.
theorem list_sup_is_upper_bound[S: JoinSemilattice](head: S, tail: List[S]) {
    list_upper_bound(List.cons(head, tail), list_sup(head, tail))
} by {
    list_sup_eq_list_join(head, tail)
    list_join_is_upper_bound(head, tail)
}

/// The list supremum is below a point exactly when the point is an upper bound for the list.
theorem list_sup_le_iff[S: JoinSemilattice](head: S, tail: List[S], c: S) {
    list_sup(head, tail) <= c = list_upper_bound(List.cons(head, tail), c)
} by {
    list_sup_lte_iff_contains_bounds(head, tail, c)
}

/// The infimum over a concatenation with a non-empty right list is the meet of the two list infima.
theorem list_inf_add_cons[S: MeetSemilattice](head: S, tail: List[S], next: S, rest: List[S]) {
    list_inf(head, tail + List.cons(next, rest)) =
    list_inf(head, tail).meet(list_inf(next, rest))
} by {
    list_inf_eq_list_meet(head, tail + List.cons(next, rest))
    list_inf_eq_list_meet(head, tail)
    list_inf_eq_list_meet(next, rest)
    list_meet_add_cons(head, tail, next, rest)
}

/// The supremum over a concatenation with a non-empty right list is the join of the two list suprema.
theorem list_sup_add_cons[S: JoinSemilattice](head: S, tail: List[S], next: S, rest: List[S]) {
    list_sup(head, tail + List.cons(next, rest)) =
    list_sup(head, tail).join(list_sup(next, rest))
} by {
    list_sup_eq_list_join(head, tail + List.cons(next, rest))
    list_sup_eq_list_join(head, tail)
    list_sup_eq_list_join(next, rest)
    list_join_add_cons(head, tail, next, rest)
}

/// The infimum over a concatenation is below the infimum of the left non-empty list.
theorem list_inf_add_cons_lte_left[S: MeetSemilattice](head: S, tail: List[S], next: S, rest: List[S]) {
    list_inf(head, tail + List.cons(next, rest)) <= list_inf(head, tail)
} by {
    list_inf_eq_list_meet(head, tail + List.cons(next, rest))
    list_inf_eq_list_meet(head, tail)
    list_meet_add_cons_lte_left(head, tail, next, rest)
}

/// The infimum over a concatenation is below the infimum of the right non-empty list.
theorem list_inf_add_cons_lte_right[S: MeetSemilattice](head: S, tail: List[S], next: S, rest: List[S]) {
    list_inf(head, tail + List.cons(next, rest)) <= list_inf(next, rest)
} by {
    list_inf_eq_list_meet(head, tail + List.cons(next, rest))
    list_inf_eq_list_meet(next, rest)
    list_meet_add_cons_lte_right(head, tail, next, rest)
}

/// The supremum of the left non-empty list is below the supremum over a concatenation.
theorem list_sup_lte_add_cons_left[S: JoinSemilattice](head: S, tail: List[S], next: S, rest: List[S]) {
    list_sup(head, tail) <= list_sup(head, tail + List.cons(next, rest))
} by {
    list_sup_eq_list_join(head, tail)
    list_sup_eq_list_join(head, tail + List.cons(next, rest))
    list_join_lte_add_cons_left(head, tail, next, rest)
}

/// The supremum of the right non-empty list is below the supremum over a concatenation.
theorem list_sup_lte_add_cons_right[S: JoinSemilattice](head: S, tail: List[S], next: S, rest: List[S]) {
    list_sup(next, rest) <= list_sup(head, tail + List.cons(next, rest))
} by {
    list_sup_eq_list_join(next, rest)
    list_sup_eq_list_join(head, tail + List.cons(next, rest))
    list_join_lte_add_cons_right(head, tail, next, rest)
}

/// A point is below the infimum over a concatenation exactly when it is below both list infima.
theorem le_list_inf_add_cons_iff[S: MeetSemilattice](c: S, head: S, tail: List[S],
        next: S, rest: List[S]) {
    c <= list_inf(head, tail + List.cons(next, rest)) =
    (c <= list_inf(head, tail) and c <= list_inf(next, rest))
} by {
    list_inf_eq_list_meet(head, tail + List.cons(next, rest))
    list_inf_eq_list_meet(head, tail)
    list_inf_eq_list_meet(next, rest)
    le_list_meet_add_cons_iff(c, head, tail, next, rest)
}

/// The supremum over a concatenation is below a point exactly when both list suprema are below it.
theorem list_sup_add_cons_le_iff[S: JoinSemilattice](head: S, tail: List[S],
        next: S, rest: List[S], c: S) {
    list_sup(head, tail + List.cons(next, rest)) <= c =
    (list_sup(head, tail) <= c and list_sup(next, rest) <= c)
} by {
    list_sup_eq_list_join(head, tail + List.cons(next, rest))
    list_sup_eq_list_join(head, tail)
    list_sup_eq_list_join(next, rest)
    list_join_add_cons_le_iff(head, tail, next, rest, c)
}

/// Appending one element takes the meet of the old list infimum and that element.
theorem list_inf_append[S: MeetSemilattice](head: S, tail: List[S], last: S) {
    list_inf(head, tail.append(last)) = list_inf(head, tail).meet(last)
} by {
    list_inf_eq_list_meet(head, tail.append(last))
    list_inf_eq_list_meet(head, tail)
    list_meet_append(head, tail, last)
}

/// Appending one element takes the join of the old list supremum and that element.
theorem list_sup_append[S: JoinSemilattice](head: S, tail: List[S], last: S) {
    list_sup(head, tail.append(last)) = list_sup(head, tail).join(last)
} by {
    list_sup_eq_list_join(head, tail.append(last))
    list_sup_eq_list_join(head, tail)
    list_join_append(head, tail, last)
}

/// The infimum after appending an element is below the previous list infimum.
theorem list_inf_append_lte_list_inf[S: MeetSemilattice](head: S, tail: List[S], last: S) {
    list_inf(head, tail.append(last)) <= list_inf(head, tail)
} by {
    list_inf_eq_list_meet(head, tail.append(last))
    list_inf_eq_list_meet(head, tail)
    list_meet_append_lte_list_meet(head, tail, last)
}

/// The infimum after appending an element is below that element.
theorem list_inf_append_lte_last[S: MeetSemilattice](head: S, tail: List[S], last: S) {
    list_inf(head, tail.append(last)) <= last
} by {
    list_inf_eq_list_meet(head, tail.append(last))
    list_meet_append_lte_last(head, tail, last)
}

/// The previous list supremum is below the supremum after appending an element.
theorem list_sup_lte_append_sup[S: JoinSemilattice](head: S, tail: List[S], last: S) {
    list_sup(head, tail) <= list_sup(head, tail.append(last))
} by {
    list_sup_eq_list_join(head, tail)
    list_sup_eq_list_join(head, tail.append(last))
    list_join_lte_append_join(head, tail, last)
}

/// The appended element is below the supremum after appending it.
theorem last_lte_list_sup_append[S: JoinSemilattice](head: S, tail: List[S], last: S) {
    last <= list_sup(head, tail.append(last))
} by {
    list_sup_eq_list_join(head, tail.append(last))
    last_lte_list_join_append(head, tail, last)
}

/// A point is below the infimum after append exactly when it is below the previous infimum and the appended element.
theorem le_list_inf_append_iff[S: MeetSemilattice](c: S, head: S, tail: List[S], last: S) {
    c <= list_inf(head, tail.append(last)) =
    (c <= list_inf(head, tail) and c <= last)
} by {
    list_inf_eq_list_meet(head, tail.append(last))
    list_inf_eq_list_meet(head, tail)
    le_list_meet_append_iff(c, head, tail, last)
}

/// The supremum after append is below a point exactly when the previous supremum and the appended element are below it.
theorem list_sup_append_le_iff[S: JoinSemilattice](head: S, tail: List[S], last: S, c: S) {
    list_sup(head, tail.append(last)) <= c =
    (list_sup(head, tail) <= c and last <= c)
} by {
    list_sup_eq_list_join(head, tail.append(last))
    list_sup_eq_list_join(head, tail)
    list_join_append_le_iff(head, tail, last, c)
}

/// The meet of a singleton list is its element.
theorem list_meet_singleton[S: MeetSemilattice](head: S) {
    list_meet(head, List.nil[S]) = head
}

/// The join of a singleton list is its element.
theorem list_join_singleton[S: JoinSemilattice](head: S) {
    list_join(head, List.nil[S]) = head
}

/// The infimum of a singleton list is its element.
theorem list_inf_singleton[S: MeetSemilattice](head: S) {
    list_inf(head, List.nil[S]) = head
} by {
    list_inf_eq_list_meet(head, List.nil[S])
    list_meet_singleton(head)
}

/// The supremum of a singleton list is its element.
theorem list_sup_singleton[S: JoinSemilattice](head: S) {
    list_sup(head, List.nil[S]) = head
} by {
    list_sup_eq_list_join(head, List.nil[S])
    list_join_singleton(head)
}

/// The meet of a cons list is the meet of the head and the tail meet.
theorem list_meet_cons[S: MeetSemilattice](head: S, next: S, rest: List[S]) {
    list_meet(head, List.cons(next, rest)) = head.meet(list_meet(next, rest))
}

/// The join of a cons list is the join of the head and the tail join.
theorem list_join_cons[S: JoinSemilattice](head: S, next: S, rest: List[S]) {
    list_join(head, List.cons(next, rest)) = head.join(list_join(next, rest))
}

/// The infimum of a cons list is the meet of the head and the tail infimum.
theorem list_inf_cons[S: MeetSemilattice](head: S, next: S, rest: List[S]) {
    list_inf(head, List.cons(next, rest)) = head.meet(list_inf(next, rest))
} by {
    list_inf_eq_list_meet(head, List.cons(next, rest))
    list_inf_eq_list_meet(next, rest)
    list_meet_cons(head, next, rest)
}

/// The supremum of a cons list is the join of the head and the tail supremum.
theorem list_sup_cons[S: JoinSemilattice](head: S, next: S, rest: List[S]) {
    list_sup(head, List.cons(next, rest)) = head.join(list_sup(next, rest))
} by {
    list_sup_eq_list_join(head, List.cons(next, rest))
    list_sup_eq_list_join(next, rest)
    list_join_cons(head, next, rest)
}

/// The meet of a two-element list is the meet of its elements.
theorem list_meet_pair[S: MeetSemilattice](a: S, b: S) {
    list_meet(a, List.cons(b, List.nil[S])) = a.meet(b)
} by {
    list_meet_cons(a, b, List.nil[S])
    list_meet_singleton(b)
}

/// The join of a two-element list is the join of its elements.
theorem list_join_pair[S: JoinSemilattice](a: S, b: S) {
    list_join(a, List.cons(b, List.nil[S])) = a.join(b)
} by {
    list_join_cons(a, b, List.nil[S])
    list_join_singleton(b)
}

/// The infimum of a two-element list is the meet of its elements.
theorem list_inf_pair[S: MeetSemilattice](a: S, b: S) {
    list_inf(a, List.cons(b, List.nil[S])) = a.meet(b)
} by {
    list_inf_cons(a, b, List.nil[S])
    list_inf_singleton(b)
}

/// The supremum of a two-element list is the join of its elements.
theorem list_sup_pair[S: JoinSemilattice](a: S, b: S) {
    list_sup(a, List.cons(b, List.nil[S])) = a.join(b)
} by {
    list_sup_cons(a, b, List.nil[S])
    list_sup_singleton(b)
}

/// A point below the head and tail infimum is below the cons-list infimum.
theorem le_list_inf_cons_of_le_head_of_le_tail[S: MeetSemilattice](c: S, head: S, next: S, rest: List[S]) {
    c <= head and c <= list_inf(next, rest) implies c <= list_inf(head, List.cons(next, rest))
} by {
    if c <= head and c <= list_inf(next, rest) {
        list_inf_cons(head, next, rest)
        lte_meet_of_bounds(c, head, list_inf(next, rest))
        c <= list_inf(head, List.cons(next, rest))
    }
}

/// A cons-list supremum is below a point when the head and tail supremum are below it.
theorem list_sup_cons_le_of_head_le_of_tail_le[S: JoinSemilattice](head: S, next: S, rest: List[S], c: S) {
    head <= c and list_sup(next, rest) <= c implies list_sup(head, List.cons(next, rest)) <= c
} by {
    if head <= c and list_sup(next, rest) <= c {
        list_sup_cons(head, next, rest)
        join_lte_of_bounds(head, list_sup(next, rest), c)
        list_sup(head, List.cons(next, rest)) <= c
    }
}

/// A point is below a cons-list infimum exactly when it is below the head and tail infimum.
theorem le_list_inf_cons_iff[S: MeetSemilattice](c: S, head: S, next: S, rest: List[S]) {
    c <= list_inf(head, List.cons(next, rest)) =
    (c <= head and c <= list_inf(next, rest))
} by {
    list_inf_cons(head, next, rest)
    lte_meet_iff(c, head, list_inf(next, rest))
}

/// A cons-list supremum is below a point exactly when the head and tail supremum are below it.
theorem list_sup_cons_le_iff[S: JoinSemilattice](head: S, next: S, rest: List[S], c: S) {
    list_sup(head, List.cons(next, rest)) <= c =
    (head <= c and list_sup(next, rest) <= c)
} by {
    list_sup_cons(head, next, rest)
    join_lte_iff(head, list_sup(next, rest), c)
}

/// A point is below the infimum of a pair exactly when it is below both elements.
theorem le_list_inf_pair_iff[S: MeetSemilattice](c: S, a: S, b: S) {
    c <= list_inf(a, List.cons(b, List.nil[S])) = (c <= a and c <= b)
} by {
    list_inf_pair(a, b)
    lte_meet_iff(c, a, b)
}

/// The supremum of a pair is below a point exactly when both elements are below it.
theorem list_sup_pair_le_iff[S: JoinSemilattice](a: S, b: S, c: S) {
    list_sup(a, List.cons(b, List.nil[S])) <= c = (a <= c and b <= c)
} by {
    list_sup_pair(a, b)
    join_lte_iff(a, b, c)
}
