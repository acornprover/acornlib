from nat import Nat, add_imp_sub, add_imp_sub_left, add_sub, divides_sub, divides_gcd,
    gcd_divides_left, gcd_divides_right, lt_or_lte, lte_and_lt,
    lt_and_lte, lte_trans, add_cancels_right

from list import List, map, map_contains, is_permutation,
    unique_same_contains_imp_permutation, map_contains_of_contains
from data.list.list_pigeonhole import locally_injective_map_is_unique
from number_theory import nat_divides_one_imp_one, coprime_residues,
    coprime_residues_unique, coprime_residues_contains_imp, coprime_residues_contains_intro

numerals Nat

/// The reflection of a residue modulo `n`.
///
/// Pairs each reduced residue with another one. On the reduced residues below `n` this is an
/// involution, which is what makes the first power sum computable by pairing.
define reflect_residue(n: Nat, a: Nat) -> Nat {
    n - a
}

/// Reflecting twice returns the original residue.
///
/// Only the bound is needed, not coprimality: natural subtraction undoes itself as long as
/// nothing was clipped at zero.
theorem reflect_residue_involution(n: Nat, a: Nat) {
    a <= n implies reflect_residue(n, reflect_residue(n, a)) = a
} by {
    if a <= n {
        let (c: Nat) satisfy {
            a + c = n
        }
        add_imp_sub_left(a, c, n)
        n - a = c
        reflect_residue(n, a) = c
        add_imp_sub(a, c, n)
        n - c = a
        reflect_residue(n, c) = a
        reflect_residue(n, reflect_residue(n, a)) = a
    }
}

/// A common divisor of the modulus and a residue divides the reflection.
theorem divides_reflect_residue(n: Nat, a: Nat, d: Nat) {
    d.divides(n) and d.divides(a) implies d.divides(reflect_residue(n, a))
} by {
    if d.divides(n) and d.divides(a) {
        divides_sub(n, a, d)
        d.divides(n - a)
        d.divides(reflect_residue(n, a))
    }
}

/// A common divisor of the modulus and the reflection divides the residue.
///
/// The reflection of the reflection is the residue, so the same argument applies in reverse.
theorem divides_of_divides_reflect(n: Nat, a: Nat, d: Nat) {
    a <= n and d.divides(n) and d.divides(reflect_residue(n, a)) implies d.divides(a)
} by {
    if a <= n and d.divides(n) and d.divides(reflect_residue(n, a)) {
        divides_reflect_residue(n, reflect_residue(n, a), d)
        d.divides(reflect_residue(n, reflect_residue(n, a)))
        reflect_residue_involution(n, a)
        reflect_residue(n, reflect_residue(n, a)) = a
        d.divides(a)
    }
}

/// The reflection of a reduced residue is a reduced residue.
///
/// A common factor of the reflection and the modulus would be a common factor of the residue
/// and the modulus, of which there is none but one.
theorem reflect_residue_coprime(n: Nat, a: Nat) {
    a <= n and a.coprime(n) implies reflect_residue(n, a).coprime(n)
} by {
    if a <= n and a.coprime(n) {
        gcd_divides_left(reflect_residue(n, a), n)
        reflect_residue(n, a).gcd(n).divides(reflect_residue(n, a))
        gcd_divides_right(reflect_residue(n, a), n)
        reflect_residue(n, a).gcd(n).divides(n)
        divides_of_divides_reflect(n, a, reflect_residue(n, a).gcd(n))
        reflect_residue(n, a).gcd(n).divides(a)
        divides_gcd(reflect_residue(n, a).gcd(n), a, n)
        reflect_residue(n, a).gcd(n).divides(a.gcd(n))
        a.coprime(n) = (a.gcd(n) = Nat.1)
        a.gcd(n) = Nat.1
        reflect_residue(n, a).gcd(n).divides(Nat.1)
        nat_divides_one_imp_one(reflect_residue(n, a).gcd(n))
        reflect_residue(n, a).gcd(n) = Nat.1
        reflect_residue(n, a).coprime(n) = (reflect_residue(n, a).gcd(n) = Nat.1)
        reflect_residue(n, a).coprime(n)
    }
}

/// The reflection of a positive residue below the modulus is again one.
///
/// The reduced residues used in the sums lie strictly between zero and the modulus, and the
/// reflection keeps them there.
theorem reflect_residue_in_range(n: Nat, a: Nat) {
    Nat.0 < a and a < n implies Nat.0 < reflect_residue(n, a) and reflect_residue(n, a) < n
} by {
    if Nat.0 < a and a < n {
        a <= n
        let (c: Nat) satisfy {
            a + c = n
        }
        add_imp_sub_left(a, c, n)
        n - a = c
        reflect_residue(n, a) = c
        if c = Nat.0 {
            a + Nat.0 = n
            a = n
            a < a
            false
        }
        c != Nat.0
        Nat.0 < c
        Nat.0 < reflect_residue(n, a)
        c <= a + c
        c <= n
        if c = n {
            a + n = n
            Nat.0 + n = n
            a + n = Nat.0 + n
            add_cancels_right(n, a, Nat.0)
            a = Nat.0
            Nat.0 < Nat.0
            false
        }
        c != n
        c < n
        reflect_residue(n, a) < n
        (Nat.0 < reflect_residue(n, a) and reflect_residue(n, a) < n)
    }
}

/// The reflection fixes a residue exactly when it is half the modulus.
///
/// This is why the pairing argument for the first power sum needs the modulus to exceed two:
/// below that the reflection has a fixed point among the reduced residues and the pairing is
/// not free.
theorem reflect_residue_fixed_point(n: Nat, a: Nat) {
    a <= n and reflect_residue(n, a) = a implies a + a = n
} by {
    if a <= n and reflect_residue(n, a) = a {
        let (c: Nat) satisfy {
            a + c = n
        }
        add_imp_sub_left(a, c, n)
        n - a = c
        reflect_residue(n, a) = c
        c = a
        a + a = n
    }
}

/// A residue that is not half the modulus is moved by the reflection.
theorem reflect_residue_moves(n: Nat, a: Nat) {
    a <= n and a + a != n implies reflect_residue(n, a) != a
} by {
    if a <= n and a + a != n {
        if reflect_residue(n, a) = a {
            reflect_residue_fixed_point(n, a)
            a + a = n
            false
        }
        reflect_residue(n, a) != a
    }
}

/// The reflection never exceeds the modulus.
theorem reflect_residue_le(n: Nat, a: Nat) {
    a <= n implies reflect_residue(n, a) <= n
} by {
    if a <= n {
        let (c: Nat) satisfy {
            a + c = n
        }
        add_imp_sub_left(a, c, n)
        n - a = c
        reflect_residue(n, a) = c
        c <= a + c
        c <= n
        reflect_residue(n, a) <= n
    }
}

/// Reflecting every reduced residue permutes the list of them, for a modulus above one.
///
/// Unique because reflection is injective on residues below the modulus, and with the same
/// members because reflection carries reduced residues to reduced residues and is its own
/// inverse. This is the hypothesis `reduced_residue_sum_reindex` takes.
///
/// The bound on the modulus is necessary, not a convenience. Modulo one the only reduced
/// residue is zero, and reflection sends it to one, which is not in the list; the two lists
/// are then `[0]` and `[1]` and no permutation relates them.
theorem reflect_permutes_coprime_residues(n: Nat) {
    Nat.1 < n implies
        is_permutation(coprime_residues(n), map(coprime_residues(n), reflect_residue(n)))
} by {
    if Nat.1 < n {
    coprime_residues_unique(n)
    coprime_residues(n).is_unique
    forall(a: Nat, b: Nat) {
        if coprime_residues(n).contains(a) and coprime_residues(n).contains(b)
            and reflect_residue(n, a) = reflect_residue(n, b) {
            coprime_residues_contains_imp(n, a)
            a < n
            a <= n
            coprime_residues_contains_imp(n, b)
            b < n
            b <= n
            reflect_residue_involution(n, a)
            reflect_residue(n, reflect_residue(n, a)) = a
            reflect_residue_involution(n, b)
            reflect_residue(n, reflect_residue(n, b)) = b
            a = b
        }
        (coprime_residues(n).contains(a) and coprime_residues(n).contains(b)
            and reflect_residue(n, a) = reflect_residue(n, b) implies a = b)
    }
    locally_injective_map_is_unique(coprime_residues(n), reflect_residue(n))
    map(coprime_residues(n), reflect_residue(n)).is_unique
    forall(y: Nat) {
        if coprime_residues(n).contains(y) {
            coprime_residues_contains_imp(n, y)
            y < n
            y <= n
            reflect_residue_involution(n, y)
            reflect_residue(n, reflect_residue(n, y)) = y
            reflect_residue_coprime(n, y)
            reflect_residue(n, y).coprime(n)
            reflect_residue_le(n, y)
            reflect_residue(n, y) <= n
            if reflect_residue(n, y) = n {
                reflect_residue(n, reflect_residue(n, y)) = reflect_residue(n, n)
                n - n = Nat.0
                reflect_residue(n, n) = Nat.0
                y = Nat.0
                coprime_residues_contains_imp(n, y)
                y.coprime(n)
                Nat.0.coprime(n)
                Nat.0.gcd(n) = Nat.1
                Nat.0.gcd(n) = n
                n = Nat.1
                Nat.1 < Nat.1
                false
            }
            reflect_residue(n, y) != n
            reflect_residue(n, y) < n
            coprime_residues_contains_intro(n, reflect_residue(n, y))
            coprime_residues(n).contains(reflect_residue(n, y))
            map_contains_of_contains(coprime_residues(n), reflect_residue(n),
                reflect_residue(n, y))
            (map(coprime_residues(n), reflect_residue(n))
                .contains(reflect_residue(n, reflect_residue(n, y))))
            map(coprime_residues(n), reflect_residue(n)).contains(y)
        }
        if map(coprime_residues(n), reflect_residue(n)).contains(y) {
            map_contains(coprime_residues(n), reflect_residue(n), y)
            exists(x: Nat) {
                coprime_residues(n).contains(x) and reflect_residue(n, x) = y
            }
            let (x: Nat) satisfy {
                coprime_residues(n).contains(x) and reflect_residue(n, x) = y
            }
            coprime_residues_contains_imp(n, x)
            x < n
            x <= n
            reflect_residue_coprime(n, x)
            reflect_residue(n, x).coprime(n)
            y.coprime(n)
            reflect_residue_involution(n, x)
            reflect_residue(n, y) = x
            reflect_residue_le(n, x)
            reflect_residue(n, x) <= n
            y <= n
            if y = n {
                reflect_residue(n, y) = n - n
                n - n = Nat.0
                x = Nat.0
                coprime_residues_contains_imp(n, x)
                x.coprime(n)
                Nat.0.coprime(n)
                Nat.0.gcd(n) = Nat.1
                Nat.0.gcd(n) = n
                n = Nat.1
                Nat.1 < Nat.1
                false
            }
            y != n
            y < n
            coprime_residues_contains_intro(n, y)
            coprime_residues(n).contains(y)
        }
        coprime_residues(n).contains(y) = map(coprime_residues(n), reflect_residue(n)).contains(y)
    }
        unique_same_contains_imp_permutation(coprime_residues(n),
            map(coprime_residues(n), reflect_residue(n)))
        is_permutation(coprime_residues(n), map(coprime_residues(n), reflect_residue(n)))
    }
}

/// A residue and its reflection add to the modulus.
///
/// The pairing identity behind the first power sum.
theorem reflect_residue_adds_to_modulus(n: Nat, a: Nat) {
    a <= n implies a + reflect_residue(n, a) = n
} by {
    if a <= n {
        let (c: Nat) satisfy {
            a + c = n
        }
        add_imp_sub_left(a, c, n)
        n - a = c
        reflect_residue(n, a) = c
        a + reflect_residue(n, a) = n
    }
}
