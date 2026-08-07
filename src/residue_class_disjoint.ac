from int import Int
from list import List
from nat import Nat, divides_trans, divides_sub, gcd_divides_left, gcd_divides_right
from pair import Pair
from zmod import int_mod_rel
from data.int.int_congruence import int_congr_mod_symm, int_congr_mod_trans,
    int_congr_mod_iff_int_mod_rel, int_congr_mod_zero_iff_divides, int_congr_mod_add_right
from number_theory import congruence_class_contains, covers_int,
    covers_int_nil_false, covers_int_cons_imp, covers_int_cons_left, covers_int_cons_right

numerals Int

/// True if no integer belongs to both congruence classes.
///
/// Stated pointwise over the integers rather than through a condition on the moduli and
/// residues, so that it composes directly with `covers_int`.
define classes_disjoint(cl1: Pair[Nat, Nat], cl2: Pair[Nat, Nat]) -> Bool {
    forall(x: Int) {
        not (congruence_class_contains(cl1, x) and congruence_class_contains(cl2, x))
    }
}

/// No integer lies in both of two disjoint classes.
theorem classes_disjoint_apply(cl1: Pair[Nat, Nat], cl2: Pair[Nat, Nat], x: Int) {
    classes_disjoint(cl1, cl2)
        implies not (congruence_class_contains(cl1, x) and congruence_class_contains(cl2, x))
} by {
    if classes_disjoint(cl1, cl2) {
        classes_disjoint(cl1, cl2) = forall(y: Int) {
            not (congruence_class_contains(cl1, y) and congruence_class_contains(cl2, y))
        }
        forall(y: Int) {
            not (congruence_class_contains(cl1, y) and congruence_class_contains(cl2, y))
        }
        not (congruence_class_contains(cl1, x) and congruence_class_contains(cl2, x))
    }
}

/// A pointwise absence of shared members is disjointness.
theorem classes_disjoint_intro(cl1: Pair[Nat, Nat], cl2: Pair[Nat, Nat]) {
    (forall(x: Int) {
        not (congruence_class_contains(cl1, x) and congruence_class_contains(cl2, x))
    }) implies classes_disjoint(cl1, cl2)
} by {
    if forall(x: Int) {
        not (congruence_class_contains(cl1, x) and congruence_class_contains(cl2, x))
    } {
        classes_disjoint(cl1, cl2) = forall(y: Int) {
            not (congruence_class_contains(cl1, y) and congruence_class_contains(cl2, y))
        }
        classes_disjoint(cl1, cl2)
    }
}

/// Disjointness is symmetric.
theorem classes_disjoint_symm(cl1: Pair[Nat, Nat], cl2: Pair[Nat, Nat]) {
    classes_disjoint(cl1, cl2) implies classes_disjoint(cl2, cl1)
} by {
    if classes_disjoint(cl1, cl2) {
        forall(x: Int) {
            classes_disjoint_apply(cl1, cl2, x)
            not (congruence_class_contains(cl1, x) and congruence_class_contains(cl2, x))
            not (congruence_class_contains(cl2, x) and congruence_class_contains(cl1, x))
        }
        classes_disjoint_intro(cl2, cl1)
        classes_disjoint(cl2, cl1)
    }
}

/// True if the classes of a system are pairwise disjoint.
///
/// Recursive on the list in the same shape as `covers_int`, so the head condition is stated
/// against everything after it and the two definitions unfold together.
define head_disjoint_from(head: Pair[Nat, Nat], rest: List[Pair[Nat, Nat]]) -> Bool {
    forall(x: Int) {
        not (congruence_class_contains(head, x) and covers_int(rest, x))
    }
}

/// The head condition, at a single integer.
theorem head_disjoint_from_apply(
    head: Pair[Nat, Nat], rest: List[Pair[Nat, Nat]], x: Int
) {
    head_disjoint_from(head, rest)
        implies not (congruence_class_contains(head, x) and covers_int(rest, x))
} by {
    if head_disjoint_from(head, rest) {
        head_disjoint_from(head, rest) = forall(y: Int) {
            not (congruence_class_contains(head, y) and covers_int(rest, y))
        }
        forall(y: Int) {
            not (congruence_class_contains(head, y) and covers_int(rest, y))
        }
        not (congruence_class_contains(head, x) and covers_int(rest, x))
    }
}

/// The pointwise condition gives the head condition.
theorem head_disjoint_from_intro(head: Pair[Nat, Nat], rest: List[Pair[Nat, Nat]]) {
    (forall(x: Int) {
        not (congruence_class_contains(head, x) and covers_int(rest, x))
    }) implies head_disjoint_from(head, rest)
} by {
    if forall(x: Int) {
        not (congruence_class_contains(head, x) and covers_int(rest, x))
    } {
        head_disjoint_from(head, rest) = forall(y: Int) {
            not (congruence_class_contains(head, y) and covers_int(rest, y))
        }
        head_disjoint_from(head, rest)
    }
}

/// True if the classes of a system are pairwise disjoint.
///
/// Recursive on the list in the same shape as `covers_int`, so the head condition is stated
/// against everything after it and the two definitions unfold together. The head condition is
/// named rather than written inline: the unfolding equation is otherwise large enough that
/// proof search will not reduce it.
define disjoint_system(system: List[Pair[Nat, Nat]]) -> Bool {
    match system {
        List.nil {
            true
        }
        List.cons(head, tail) {
            head_disjoint_from(head, tail) and disjoint_system(tail)
        }
    }
}

/// A disjoint cons system has its head disjoint from the rest.
theorem disjoint_system_cons_head_part(
    head: Pair[Nat, Nat], tail: List[Pair[Nat, Nat]]
) {
    disjoint_system(List.cons(head, tail)) implies head_disjoint_from(head, tail)
} by {
    if disjoint_system(List.cons(head, tail)) {
        head_disjoint_from(head, tail)
    }
}

/// A disjoint cons system has a disjoint tail.
theorem disjoint_system_cons_tail_part(
    head: Pair[Nat, Nat], tail: List[Pair[Nat, Nat]]
) {
    disjoint_system(List.cons(head, tail)) implies disjoint_system(tail)
} by {
    if disjoint_system(List.cons(head, tail)) {
        disjoint_system(tail)
    }
}

/// The empty system is disjoint.
theorem disjoint_system_nil {
    disjoint_system(List.nil[Pair[Nat, Nat]])
}

/// A disjoint system stays disjoint after dropping its first class.
///
/// The remaining classes were already pairwise disjoint, since the condition on the head was
/// stated separately from the condition on the tail.
theorem disjoint_system_tail(head: Pair[Nat, Nat], tail: List[Pair[Nat, Nat]]) {
    disjoint_system(List.cons(head, tail)) implies disjoint_system(tail)
} by {
    if disjoint_system(List.cons(head, tail)) {
        disjoint_system_cons_tail_part(head, tail)
        disjoint_system(tail)
    }
}

/// The head of a disjoint system shares no member with the rest.
theorem disjoint_system_head(head: Pair[Nat, Nat], tail: List[Pair[Nat, Nat]], x: Int) {
    disjoint_system(List.cons(head, tail))
        implies not (congruence_class_contains(head, x) and covers_int(tail, x))
} by {
    if disjoint_system(List.cons(head, tail)) {
        disjoint_system_cons_head_part(head, tail)
        head_disjoint_from(head, tail)
        head_disjoint_from_apply(head, tail, x)
        not (congruence_class_contains(head, x) and covers_int(tail, x))
    }
}

/// The two conditions give a disjoint system.
theorem disjoint_system_cons_intro(head: Pair[Nat, Nat], tail: List[Pair[Nat, Nat]]) {
    head_disjoint_from(head, tail) and disjoint_system(tail)
        implies disjoint_system(List.cons(head, tail))
} by {
    if head_disjoint_from(head, tail) and disjoint_system(tail) {
        disjoint_system(List.cons(head, tail))
    }
}

/// A member of a class is congruent to the residue modulo the modulus.
theorem congruence_class_contains_eq(cl: Pair[Nat, Nat], x: Int) {
    congruence_class_contains(cl, x) = int_mod_rel(cl.first, x, Int.from_nat(cl.second))
}

/// The modulus of a class divides the difference between a member and the residue.
theorem class_modulus_divides_difference(cl: Pair[Nat, Nat], x: Int) {
    congruence_class_contains(cl, x)
        implies Int.from_nat(cl.first).divides(x - Int.from_nat(cl.second))
} by {
    if congruence_class_contains(cl, x) {
        congruence_class_contains_eq(cl, x)
        int_mod_rel(cl.first, x, Int.from_nat(cl.second))
        int_congr_mod_iff_int_mod_rel(x, Int.from_nat(cl.second), cl.first)
        x.congr_mod(Int.from_nat(cl.second), cl.first)
        x.congr_mod(Int.from_nat(cl.second), cl.first) = int_mod_rel(cl.first, x, Int.from_nat(cl.second))
        int_congr_mod_add_right(x, Int.from_nat(cl.second), -Int.from_nat(cl.second), cl.first)
        (x + -Int.from_nat(cl.second)).congr_mod(
            Int.from_nat(cl.second) + -Int.from_nat(cl.second), cl.first)
        Int.from_nat(cl.second) + -Int.from_nat(cl.second) = Int.0
        x + -Int.from_nat(cl.second) = x - Int.from_nat(cl.second)
        (x - Int.from_nat(cl.second)).congr_mod(Int.0, cl.first)
        int_congr_mod_zero_iff_divides(x - Int.from_nat(cl.second), cl.first)
        Int.from_nat(cl.first).divides(x - Int.from_nat(cl.second))
    }
}

/// Two classes meeting force each modulus to divide the difference from the shared member.
///
/// This is the elementary half of the disjointness criterion: a common member is congruent to
/// both residues, so the gap between the residues is a multiple of each modulus, hence of
/// their greatest common divisor.
theorem meeting_classes_moduli_divide(
    cl1: Pair[Nat, Nat], cl2: Pair[Nat, Nat], x: Int
) {
    congruence_class_contains(cl1, x) and congruence_class_contains(cl2, x)
        implies Int.from_nat(cl1.first).divides(x - Int.from_nat(cl1.second))
            and Int.from_nat(cl2.first).divides(x - Int.from_nat(cl2.second))
} by {
    if congruence_class_contains(cl1, x) and congruence_class_contains(cl2, x) {
        class_modulus_divides_difference(cl1, x)
        Int.from_nat(cl1.first).divides(x - Int.from_nat(cl1.second))
        class_modulus_divides_difference(cl2, x)
        Int.from_nat(cl2.first).divides(x - Int.from_nat(cl2.second))
        (Int.from_nat(cl1.first).divides(x - Int.from_nat(cl1.second))
            and Int.from_nat(cl2.first).divides(x - Int.from_nat(cl2.second)))
    }
}

/// Classes with the same modulus and different residues are disjoint.
///
/// A common member would make the two residues congruent modulo the shared modulus, so a
/// system built on one modulus is disjoint exactly when its residues are distinct.
theorem same_modulus_classes_disjoint(cl1: Pair[Nat, Nat], cl2: Pair[Nat, Nat]) {
    cl1.first = cl2.first
        and not int_mod_rel(cl1.first, Int.from_nat(cl1.second), Int.from_nat(cl2.second))
        implies classes_disjoint(cl1, cl2)
} by {
    if cl1.first = cl2.first
        and not int_mod_rel(cl1.first, Int.from_nat(cl1.second), Int.from_nat(cl2.second)) {
        forall(x: Int) {
            if congruence_class_contains(cl1, x) and congruence_class_contains(cl2, x) {
                congruence_class_contains_eq(cl1, x)
                int_mod_rel(cl1.first, x, Int.from_nat(cl1.second))
                congruence_class_contains_eq(cl2, x)
                int_mod_rel(cl2.first, x, Int.from_nat(cl2.second))
                int_mod_rel(cl1.first, x, Int.from_nat(cl2.second))
                int_congr_mod_iff_int_mod_rel(x, Int.from_nat(cl1.second), cl1.first)
                x.congr_mod(Int.from_nat(cl1.second), cl1.first)
                int_congr_mod_symm(x, Int.from_nat(cl1.second), cl1.first)
                Int.from_nat(cl1.second).congr_mod(x, cl1.first)
                int_congr_mod_iff_int_mod_rel(x, Int.from_nat(cl2.second), cl1.first)
                x.congr_mod(Int.from_nat(cl2.second), cl1.first)
                int_congr_mod_trans(Int.from_nat(cl1.second), x, Int.from_nat(cl2.second),
                    cl1.first)
                Int.from_nat(cl1.second).congr_mod(Int.from_nat(cl2.second), cl1.first)
                int_congr_mod_iff_int_mod_rel(Int.from_nat(cl1.second),
                    Int.from_nat(cl2.second), cl1.first)
                int_mod_rel(cl1.first, Int.from_nat(cl1.second), Int.from_nat(cl2.second))
                false
            }
            not (congruence_class_contains(cl1, x) and congruence_class_contains(cl2, x))
        }
        classes_disjoint_intro(cl1, cl2)
        classes_disjoint(cl1, cl2)
    }
}

/// A disjoint system covering an integer covers it through exactly one class.
///
/// If the head covers it, nothing later does; if something later covers it, the head does
/// not. This is what makes counting over a disjoint system well defined.
theorem disjoint_system_unique_cover(
    head: Pair[Nat, Nat], tail: List[Pair[Nat, Nat]], x: Int
) {
    disjoint_system(List.cons(head, tail)) and congruence_class_contains(head, x)
        implies not covers_int(tail, x)
} by {
    if disjoint_system(List.cons(head, tail)) and congruence_class_contains(head, x) {
        disjoint_system_head(head, tail, x)
        not (congruence_class_contains(head, x) and covers_int(tail, x))
        not covers_int(tail, x)
    }
}

/// In a disjoint system, a class later in the list excludes the head.
theorem disjoint_system_tail_excludes_head(
    head: Pair[Nat, Nat], tail: List[Pair[Nat, Nat]], x: Int
) {
    disjoint_system(List.cons(head, tail)) and covers_int(tail, x)
        implies not congruence_class_contains(head, x)
} by {
    if disjoint_system(List.cons(head, tail)) and covers_int(tail, x) {
        disjoint_system_head(head, tail, x)
        not (congruence_class_contains(head, x) and covers_int(tail, x))
        not congruence_class_contains(head, x)
    }
}
