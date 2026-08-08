from algebra.add_comm_group import AddCommGroup
from pair import Pair, pair_ext, pair_new_first, pair_new_second
from affine_space import AffineSpace, affine_space_product,
    affine_space_product_vadd_apply, affine_space_product_vsub_apply
from affine_map import AffineMap, affine_map_constraint, affine_map_new_apply,
    affine_map_new_src, affine_map_new_dst, affine_map_preserves, affine_map_ext,
    affine_map_id, affine_map_id_apply, affine_map_id_src, affine_map_id_dst,
    affine_map_compose, affine_map_compose_src, affine_map_compose_dst,
    affine_map_compose_apply, affine_map_compose_some, affine_map_preimage,
    affine_map_preimage_some, affine_map_preimage_space,
    affine_map_preimage_eq_of_contains, affine_map_preimage_some_of_eq
from affine_subspace import AffineSubspace, affine_subspace_univ,
    affine_subspace_univ_space,
    affine_subspace_product, affine_subspace_product_space,
    affine_subspace_product_univ_right_contains_of_eq,
    affine_subspace_product_univ_left_contains_of_eq

/// The first coordinate function on a product of point types.
define affine_product_fst[P, Q](p: Pair[P, Q]) -> P {
    p.first
}

/// The second coordinate function on a product of point types.
define affine_product_snd[P, Q](p: Pair[P, Q]) -> Q {
    p.second
}

/// The first coordinate function preserves affine combinations.
theorem affine_map_product_fst_constraint[V: AddCommGroup, W: AddCommGroup, P, Q](
    a: AffineSpace[V, P], b: AffineSpace[W, Q]) {
    affine_map_constraint(
        affine_space_product(a, b), a, affine_product_fst[P, Q])
} by {
    forall(p: Pair[P, Q], q: Pair[P, Q], r: Pair[P, Q]) {
        affine_space_product_vsub_apply(a, b, p, q)
        let difference = affine_space_product(a, b).vsub(p, q)
        difference = Pair.new(a.vsub(p.first, q.first), b.vsub(p.second, q.second))
        pair_new_first(a.vsub(p.first, q.first), b.vsub(p.second, q.second))
        difference.first = a.vsub(p.first, q.first)
        affine_space_product_vadd_apply(a, b, difference, r)
        let translated = affine_space_product(a, b).vadd(difference, r)
        translated = Pair.new(
            a.vadd(difference.first, r.first), b.vadd(difference.second, r.second))
        pair_new_first(
            a.vadd(difference.first, r.first), b.vadd(difference.second, r.second))
        translated.first = a.vadd(difference.first, r.first)
        translated.first = a.vadd(a.vsub(p.first, q.first), r.first)
        affine_product_fst(translated) = translated.first
        affine_product_fst(p) = p.first
        affine_product_fst(q) = q.first
        affine_product_fst(r) = r.first
        affine_product_fst(affine_space_product(a, b).vadd(
            affine_space_product(a, b).vsub(p, q), r)) =
            a.vadd(a.vsub(affine_product_fst(p), affine_product_fst(q)),
                affine_product_fst(r))
    }
}

/// The second coordinate function preserves affine combinations.
theorem affine_map_product_snd_constraint[V: AddCommGroup, W: AddCommGroup, P, Q](
    a: AffineSpace[V, P], b: AffineSpace[W, Q]) {
    affine_map_constraint(
        affine_space_product(a, b), b, affine_product_snd[P, Q])
} by {
    forall(p: Pair[P, Q], q: Pair[P, Q], r: Pair[P, Q]) {
        affine_space_product_vsub_apply(a, b, p, q)
        let difference = affine_space_product(a, b).vsub(p, q)
        difference = Pair.new(a.vsub(p.first, q.first), b.vsub(p.second, q.second))
        pair_new_second(a.vsub(p.first, q.first), b.vsub(p.second, q.second))
        difference.second = b.vsub(p.second, q.second)
        affine_space_product_vadd_apply(a, b, difference, r)
        let translated = affine_space_product(a, b).vadd(difference, r)
        translated = Pair.new(
            a.vadd(difference.first, r.first), b.vadd(difference.second, r.second))
        pair_new_second(
            a.vadd(difference.first, r.first), b.vadd(difference.second, r.second))
        translated.second = b.vadd(difference.second, r.second)
        translated.second = b.vadd(b.vsub(p.second, q.second), r.second)
        affine_product_snd(translated) = translated.second
        affine_product_snd(p) = p.second
        affine_product_snd(q) = q.second
        affine_product_snd(r) = r.second
        affine_product_snd(affine_space_product(a, b).vadd(
            affine_space_product(a, b).vsub(p, q), r)) =
            b.vadd(b.vsub(affine_product_snd(p), affine_product_snd(q)),
                affine_product_snd(r))
    }
}

/// The first coordinate projection from a product affine space.
let affine_map_product_fst[V: AddCommGroup, W: AddCommGroup, P, Q](a: AffineSpace[V, P], b: AffineSpace[W, Q]) -> result: AffineMap[Pair[V, W], Pair[P, Q], V, P] satisfy {
    AffineMap.new(
        affine_space_product(a, b), a, affine_product_fst[P, Q]) =
        Option.some(result)
} by {
    affine_map_product_fst_constraint(a, b)
}

/// The second coordinate projection from a product affine space.
let affine_map_product_snd[V: AddCommGroup, W: AddCommGroup, P, Q](a: AffineSpace[V, P], b: AffineSpace[W, Q]) -> result: AffineMap[Pair[V, W], Pair[P, Q], W, Q] satisfy {
    AffineMap.new(
        affine_space_product(a, b), b, affine_product_snd[P, Q]) =
        Option.some(result)
} by {
    affine_map_product_snd_constraint(a, b)
}

/// The first coordinate projection evaluates to the first coordinate.
theorem affine_map_product_fst_apply[V: AddCommGroup, W: AddCommGroup, P, Q](
    a: AffineSpace[V, P], b: AffineSpace[W, Q], p: Pair[P, Q]) {
    affine_map_product_fst(a, b).to_fun(p) = p.first
} by {
    AffineMap.new(
        affine_space_product(a, b), a, affine_product_fst[P, Q]) =
        Option.some(affine_map_product_fst(a, b))
    affine_map_new_apply(
        affine_space_product(a, b), a, affine_product_fst[P, Q],
        affine_map_product_fst(a, b), p)
}

/// The second coordinate projection evaluates to the second coordinate.
theorem affine_map_product_snd_apply[V: AddCommGroup, W: AddCommGroup, P, Q](
    a: AffineSpace[V, P], b: AffineSpace[W, Q], p: Pair[P, Q]) {
    affine_map_product_snd(a, b).to_fun(p) = p.second
} by {
    AffineMap.new(
        affine_space_product(a, b), b, affine_product_snd[P, Q]) =
        Option.some(affine_map_product_snd(a, b))
    affine_map_new_apply(
        affine_space_product(a, b), b, affine_product_snd[P, Q],
        affine_map_product_snd(a, b), p)
}

/// The source of the first coordinate projection is the product affine space.
theorem affine_map_product_fst_src[V: AddCommGroup, W: AddCommGroup, P, Q](
    a: AffineSpace[V, P], b: AffineSpace[W, Q]) {
    affine_map_product_fst(a, b).src = affine_space_product(a, b)
}

/// The target of the first coordinate projection is the first affine space.
theorem affine_map_product_fst_dst[V: AddCommGroup, W: AddCommGroup, P, Q](
    a: AffineSpace[V, P], b: AffineSpace[W, Q]) {
    affine_map_product_fst(a, b).dst = a
}

/// The source of the second coordinate projection is the product affine space.
theorem affine_map_product_snd_src[V: AddCommGroup, W: AddCommGroup, P, Q](
    a: AffineSpace[V, P], b: AffineSpace[W, Q]) {
    affine_map_product_snd(a, b).src = affine_space_product(a, b)
}

/// The target of the second coordinate projection is the second affine space.
theorem affine_map_product_snd_dst[V: AddCommGroup, W: AddCommGroup, P, Q](
    a: AffineSpace[V, P], b: AffineSpace[W, Q]) {
    affine_map_product_snd(a, b).dst = b
}

/// A first-coordinate preimage subspace is the product with the full
/// second-coordinate affine space.
theorem affine_map_product_fst_preimage_eq[V: AddCommGroup, W: AddCommGroup, P, Q](
    a: AffineSpace[V, P], b: AffineSpace[W, Q],
    s: AffineSubspace[V, P],
    pre: AffineSubspace[Pair[V, W], Pair[P, Q]]
) {
    s.space = a
        and affine_map_preimage(affine_map_product_fst(a, b), s) =
            Option.some(pre)
        implies pre = affine_subspace_product(s, affine_subspace_univ(b))
} by {
    if s.space = a
        and affine_map_preimage(affine_map_product_fst(a, b), s) =
            Option.some(pre) {
        let f = affine_map_product_fst(a, b)
        let univ_b = affine_subspace_univ(b)
        let product = affine_subspace_product(s, univ_b)
        affine_map_preimage(f, s) = Option.some(pre)
        affine_map_preimage_space(f, s, pre)
        affine_map_product_fst_src(a, b)
        affine_subspace_univ_space(b)
        affine_subspace_product_space(s, univ_b)
        pre.space = affine_space_product(a, b)
        product.space = affine_space_product(a, b)
        pre.space = product.space
        forall(x: Pair[P, Q]) {
            affine_map_product_fst_apply(a, b, x)
            f.to_fun(x) = x.first
            affine_subspace_product_univ_right_contains_of_eq(
                s, b, x, f.to_fun(x))
            product.contains(x) = s.contains(f.to_fun(x))
        }
        affine_map_preimage_eq_of_contains(f, s, pre, product)
        pre = product
        pre = affine_subspace_product(s, affine_subspace_univ(b))
    }
}

/// The preimage of a first-coordinate subspace is its product with the full
/// second-coordinate affine space.
theorem affine_map_product_fst_preimage[V: AddCommGroup, W: AddCommGroup, P, Q](
    a: AffineSpace[V, P], b: AffineSpace[W, Q],
    s: AffineSubspace[V, P]
) {
    s.space = a implies
        affine_map_preimage(affine_map_product_fst(a, b), s) =
            Option.some(affine_subspace_product(s, affine_subspace_univ(b)))
} by {
    if s.space = a {
        let f = affine_map_product_fst(a, b)
        affine_map_product_fst_dst(a, b)
        affine_map_preimage_some(f, s)
        let pre: AffineSubspace[Pair[V, W], Pair[P, Q]] satisfy {
            affine_map_preimage(f, s) = Option.some(pre)
        }
        affine_map_product_fst_preimage_eq(a, b, s, pre)
        affine_map_preimage_some_of_eq(
            f, s, pre,
            affine_subspace_product(s, affine_subspace_univ(b)))
        affine_map_preimage(f, s) =
            Option.some(affine_subspace_product(s, affine_subspace_univ(b)))
        affine_map_preimage(affine_map_product_fst(a, b), s) =
            Option.some(affine_subspace_product(s, affine_subspace_univ(b)))
    }
}

/// A second-coordinate preimage subspace is the product of the full
/// first-coordinate affine space with that subspace.
theorem affine_map_product_snd_preimage_eq[V: AddCommGroup, W: AddCommGroup, P, Q](
    a: AffineSpace[V, P], b: AffineSpace[W, Q],
    s: AffineSubspace[W, Q],
    pre: AffineSubspace[Pair[V, W], Pair[P, Q]]
) {
    s.space = b
        and affine_map_preimage(affine_map_product_snd(a, b), s) =
            Option.some(pre)
        implies pre = affine_subspace_product(affine_subspace_univ(a), s)
} by {
    if s.space = b
        and affine_map_preimage(affine_map_product_snd(a, b), s) =
            Option.some(pre) {
        let f = affine_map_product_snd(a, b)
        let univ_a = affine_subspace_univ(a)
        let product = affine_subspace_product(univ_a, s)
        affine_map_preimage(f, s) = Option.some(pre)
        affine_map_preimage_space(f, s, pre)
        affine_map_product_snd_src(a, b)
        affine_subspace_univ_space(a)
        affine_subspace_product_space(univ_a, s)
        pre.space = affine_space_product(a, b)
        product.space = affine_space_product(a, b)
        pre.space = product.space
        forall(x: Pair[P, Q]) {
            affine_map_product_snd_apply(a, b, x)
            f.to_fun(x) = x.second
            affine_subspace_product_univ_left_contains_of_eq(
                a, s, x, f.to_fun(x))
            product.contains(x) = s.contains(f.to_fun(x))
        }
        affine_map_preimage_eq_of_contains(f, s, pre, product)
        pre = product
        pre = affine_subspace_product(affine_subspace_univ(a), s)
    }
}

/// The preimage of a second-coordinate subspace is the product of the full
/// first-coordinate affine space with that subspace.
theorem affine_map_product_snd_preimage[V: AddCommGroup, W: AddCommGroup, P, Q](
    a: AffineSpace[V, P], b: AffineSpace[W, Q],
    s: AffineSubspace[W, Q]
) {
    s.space = b implies
        affine_map_preimage(affine_map_product_snd(a, b), s) =
            Option.some(affine_subspace_product(affine_subspace_univ(a), s))
} by {
    if s.space = b {
        let f = affine_map_product_snd(a, b)
        affine_map_product_snd_dst(a, b)
        affine_map_preimage_some(f, s)
        let pre: AffineSubspace[Pair[V, W], Pair[P, Q]] satisfy {
            affine_map_preimage(f, s) = Option.some(pre)
        }
        affine_map_product_snd_preimage_eq(a, b, s, pre)
        affine_map_preimage_some_of_eq(
            f, s, pre,
            affine_subspace_product(affine_subspace_univ(a), s))
        affine_map_preimage(f, s) =
            Option.some(affine_subspace_product(affine_subspace_univ(a), s))
        affine_map_preimage(affine_map_product_snd(a, b), s) =
            Option.some(affine_subspace_product(affine_subspace_univ(a), s))
    }
}

/// The componentwise function determined by two affine maps.
define affine_map_product_fun[V1: AddCommGroup, P1, V2: AddCommGroup, P2, W1: AddCommGroup, Q1, W2: AddCommGroup, Q2](
    f: AffineMap[V1, P1, V2, P2], g: AffineMap[W1, Q1, W2, Q2],
    p: Pair[P1, Q1]
) -> Pair[P2, Q2] {
    Pair.new(f.to_fun(p.first), g.to_fun(p.second))
}

/// The componentwise product function evaluates as a pair of images.
theorem affine_map_product_fun_apply[V1: AddCommGroup, P1, V2: AddCommGroup, P2, W1: AddCommGroup, Q1, W2: AddCommGroup, Q2](
    f: AffineMap[V1, P1, V2, P2], g: AffineMap[W1, Q1, W2, Q2],
    p: Pair[P1, Q1]
) {
    affine_map_product_fun(f, g, p) =
        Pair.new(f.to_fun(p.first), g.to_fun(p.second))
}

/// Componentwise application of two affine maps preserves affine combinations.
theorem affine_map_product_constraint[V1: AddCommGroup, P1, V2: AddCommGroup, P2, W1: AddCommGroup, Q1, W2: AddCommGroup, Q2](
    f: AffineMap[V1, P1, V2, P2], g: AffineMap[W1, Q1, W2, Q2]
) {
    affine_map_constraint(
        affine_space_product(f.src, g.src), affine_space_product(f.dst, g.dst),
        affine_map_product_fun(f, g))
} by {
    let source = affine_space_product(f.src, g.src)
    let target = affine_space_product(f.dst, g.dst)
    let map = affine_map_product_fun(f, g)
    affine_map_constraint(source, target, map) =
        forall(p: Pair[P1, Q1], q: Pair[P1, Q1], r: Pair[P1, Q1]) {
            map(source.vadd(source.vsub(p, q), r)) =
                target.vadd(target.vsub(map(p), map(q)), map(r))
        }
    forall(p: Pair[P1, Q1], q: Pair[P1, Q1], r: Pair[P1, Q1]) {
        affine_space_product_vsub_apply(f.src, g.src, p, q)
        let source_difference = source.vsub(p, q)
        source_difference = Pair.new(
            f.src.vsub(p.first, q.first), g.src.vsub(p.second, q.second))
        pair_new_first(
            f.src.vsub(p.first, q.first), g.src.vsub(p.second, q.second))
        pair_new_second(
            f.src.vsub(p.first, q.first), g.src.vsub(p.second, q.second))
        affine_space_product_vadd_apply(f.src, g.src, source_difference, r)
        let source_combination = source.vadd(source_difference, r)
        source_combination = Pair.new(
            f.src.vadd(source_difference.first, r.first),
            g.src.vadd(source_difference.second, r.second))
        pair_new_first(
            f.src.vadd(source_difference.first, r.first),
            g.src.vadd(source_difference.second, r.second))
        pair_new_second(
            f.src.vadd(source_difference.first, r.first),
            g.src.vadd(source_difference.second, r.second))
        source_combination.first =
            f.src.vadd(f.src.vsub(p.first, q.first), r.first)
        source_combination.second =
            g.src.vadd(g.src.vsub(p.second, q.second), r.second)

        affine_map_preserves(f, p.first, q.first, r.first)
        affine_map_preserves(g, p.second, q.second, r.second)

        let mp = map(p)
        let mq = map(q)
        let mr = map(r)
        affine_map_product_fun_apply(f, g, p)
        affine_map_product_fun_apply(f, g, q)
        affine_map_product_fun_apply(f, g, r)
        mp = Pair.new(f.to_fun(p.first), g.to_fun(p.second))
        mq = Pair.new(f.to_fun(q.first), g.to_fun(q.second))
        mr = Pair.new(f.to_fun(r.first), g.to_fun(r.second))
        pair_new_first(f.to_fun(p.first), g.to_fun(p.second))
        pair_new_second(f.to_fun(p.first), g.to_fun(p.second))
        pair_new_first(f.to_fun(q.first), g.to_fun(q.second))
        pair_new_second(f.to_fun(q.first), g.to_fun(q.second))
        pair_new_first(f.to_fun(r.first), g.to_fun(r.second))
        pair_new_second(f.to_fun(r.first), g.to_fun(r.second))

        affine_space_product_vsub_apply(f.dst, g.dst, mp, mq)
        let target_difference = target.vsub(mp, mq)
        target_difference = Pair.new(
            f.dst.vsub(mp.first, mq.first), g.dst.vsub(mp.second, mq.second))
        pair_new_first(
            f.dst.vsub(mp.first, mq.first), g.dst.vsub(mp.second, mq.second))
        pair_new_second(
            f.dst.vsub(mp.first, mq.first), g.dst.vsub(mp.second, mq.second))
        affine_space_product_vadd_apply(f.dst, g.dst, target_difference, mr)
        let target_combination = target.vadd(target_difference, mr)
        target_combination = Pair.new(
            f.dst.vadd(target_difference.first, mr.first),
            g.dst.vadd(target_difference.second, mr.second))

        let mapped_combination = map(source_combination)
        affine_map_product_fun_apply(f, g, source_combination)
        mapped_combination = Pair.new(
            f.to_fun(source_combination.first),
            g.to_fun(source_combination.second))
        pair_new_first(
            f.to_fun(source_combination.first),
            g.to_fun(source_combination.second))
        pair_new_second(
            f.to_fun(source_combination.first),
            g.to_fun(source_combination.second))
        pair_new_first(
            f.dst.vadd(target_difference.first, mr.first),
            g.dst.vadd(target_difference.second, mr.second))
        pair_new_second(
            f.dst.vadd(target_difference.first, mr.first),
            g.dst.vadd(target_difference.second, mr.second))
        mapped_combination.first = target_combination.first
        mapped_combination.second = target_combination.second
        pair_ext(mapped_combination, target_combination)
        map(source.vadd(source.vsub(p, q), r)) =
            target.vadd(target.vsub(map(p), map(q)), map(r))
    }
    affine_map_constraint(source, target, map)
}

/// The componentwise product of two affine maps.
let affine_map_product[V1: AddCommGroup, P1, V2: AddCommGroup, P2, W1: AddCommGroup, Q1, W2: AddCommGroup, Q2](f: AffineMap[V1, P1, V2, P2], g: AffineMap[W1, Q1, W2, Q2]) -> result: AffineMap[Pair[V1, W1], Pair[P1, Q1], Pair[V2, W2], Pair[P2, Q2]] satisfy {
    AffineMap.new(
        affine_space_product(f.src, g.src), affine_space_product(f.dst, g.dst),
        affine_map_product_fun(f, g)) = Option.some(result)
} by {
    affine_map_product_constraint(f, g)
}

/// The product of affine maps evaluates componentwise.
theorem affine_map_product_apply[V1: AddCommGroup, P1, V2: AddCommGroup, P2, W1: AddCommGroup, Q1, W2: AddCommGroup, Q2](
    f: AffineMap[V1, P1, V2, P2], g: AffineMap[W1, Q1, W2, Q2],
    p: Pair[P1, Q1]
) {
    affine_map_product(f, g).to_fun(p) =
        Pair.new(f.to_fun(p.first), g.to_fun(p.second))
} by {
    AffineMap.new(
        affine_space_product(f.src, g.src), affine_space_product(f.dst, g.dst),
        affine_map_product_fun(f, g)) = Option.some(affine_map_product(f, g))
    affine_map_new_apply(
        affine_space_product(f.src, g.src), affine_space_product(f.dst, g.dst),
        affine_map_product_fun(f, g), affine_map_product(f, g), p)
}

/// The source of a product of affine maps is the product of their sources.
theorem affine_map_product_src[V1: AddCommGroup, P1, V2: AddCommGroup, P2, W1: AddCommGroup, Q1, W2: AddCommGroup, Q2](
    f: AffineMap[V1, P1, V2, P2], g: AffineMap[W1, Q1, W2, Q2]
) {
    affine_map_product(f, g).src = affine_space_product(f.src, g.src)
} by {
    AffineMap.new(
        affine_space_product(f.src, g.src), affine_space_product(f.dst, g.dst),
        affine_map_product_fun(f, g)) = Option.some(affine_map_product(f, g))
    affine_map_new_src(
        affine_space_product(f.src, g.src), affine_space_product(f.dst, g.dst),
        affine_map_product_fun(f, g), affine_map_product(f, g))
}

/// The target of a product of affine maps is the product of their targets.
theorem affine_map_product_dst[V1: AddCommGroup, P1, V2: AddCommGroup, P2, W1: AddCommGroup, Q1, W2: AddCommGroup, Q2](
    f: AffineMap[V1, P1, V2, P2], g: AffineMap[W1, Q1, W2, Q2]
) {
    affine_map_product(f, g).dst = affine_space_product(f.dst, g.dst)
} by {
    AffineMap.new(
        affine_space_product(f.src, g.src), affine_space_product(f.dst, g.dst),
        affine_map_product_fun(f, g)) = Option.some(affine_map_product(f, g))
    affine_map_new_dst(
        affine_space_product(f.src, g.src), affine_space_product(f.dst, g.dst),
        affine_map_product_fun(f, g), affine_map_product(f, g))
}

/// The product of identity affine maps is the identity on the product.
theorem affine_map_product_id[V: AddCommGroup, W: AddCommGroup, P, Q](
    a: AffineSpace[V, P], b: AffineSpace[W, Q]
) {
    affine_map_product(affine_map_id(a), affine_map_id(b)) =
        affine_map_id(affine_space_product(a, b))
} by {
    let lhs = affine_map_product(affine_map_id(a), affine_map_id(b))
    let rhs = affine_map_id(affine_space_product(a, b))
    affine_map_product_src(affine_map_id(a), affine_map_id(b))
    affine_map_id_src(a)
    affine_map_id_src(b)
    lhs.src = affine_space_product(a, b)
    affine_map_id_src(affine_space_product(a, b))
    rhs.src = affine_space_product(a, b)
    lhs.src = rhs.src

    affine_map_product_dst(affine_map_id(a), affine_map_id(b))
    affine_map_id_dst(a)
    affine_map_id_dst(b)
    lhs.dst = affine_space_product(a, b)
    affine_map_id_dst(affine_space_product(a, b))
    rhs.dst = affine_space_product(a, b)
    lhs.dst = rhs.dst

    forall(p: Pair[P, Q]) {
        affine_map_product_apply(affine_map_id(a), affine_map_id(b), p)
        affine_map_id_apply(a, p.first)
        affine_map_id_apply(b, p.second)
        pair_new_first(p.first, p.second)
        pair_new_second(p.first, p.second)
        affine_map_id_apply(affine_space_product(a, b), p)
        lhs.to_fun(p) = rhs.to_fun(p)
    }
    affine_map_ext(lhs, rhs)
    lhs = rhs
}

/// Products of affine maps preserve componentwise composition.
theorem affine_map_product_compose[V1: AddCommGroup, P1, V2: AddCommGroup, P2, V3: AddCommGroup, P3, W1: AddCommGroup, Q1, W2: AddCommGroup, Q2, W3: AddCommGroup, Q3](
    f1: AffineMap[V1, P1, V2, P2], f2: AffineMap[V2, P2, V3, P3],
    g1: AffineMap[W1, Q1, W2, Q2], g2: AffineMap[W2, Q2, W3, Q3],
    fcomp: AffineMap[V1, P1, V3, P3], gcomp: AffineMap[W1, Q1, W3, Q3]
) {
    affine_map_compose(f1, f2) = Option.some(fcomp)
    and affine_map_compose(g1, g2) = Option.some(gcomp)
    implies affine_map_compose(
        affine_map_product(f1, g1), affine_map_product(f2, g2)) =
        Option.some(affine_map_product(fcomp, gcomp))
} by {
    if affine_map_compose(f1, f2) = Option.some(fcomp)
        and affine_map_compose(g1, g2) = Option.some(gcomp) {
        affine_map_compose_src(f1, f2, fcomp)
        affine_map_compose_dst(f1, f2, fcomp)
        affine_map_compose_src(g1, g2, gcomp)
        affine_map_compose_dst(g1, g2, gcomp)

        let first_product = affine_map_product(f1, g1)
        let second_product = affine_map_product(f2, g2)
        let component_composition = affine_map_product(fcomp, gcomp)
        affine_map_product_src(f1, g1)
        affine_map_product_dst(f2, g2)
        affine_map_product_src(fcomp, gcomp)
        affine_map_product_dst(fcomp, gcomp)
        component_composition.src = first_product.src
        component_composition.dst = second_product.dst

        let composed_fun: Pair[P1, Q1] -> Pair[P3, Q3] = function(p: Pair[P1, Q1]) {
            second_product.to_fun(first_product.to_fun(p))
        }
        forall(p: Pair[P1, Q1]) {
            affine_map_product_apply(f1, g1, p)
            let middle = first_product.to_fun(p)
            middle = Pair.new(f1.to_fun(p.first), g1.to_fun(p.second))
            pair_new_first(f1.to_fun(p.first), g1.to_fun(p.second))
            pair_new_second(f1.to_fun(p.first), g1.to_fun(p.second))
            affine_map_product_apply(f2, g2, middle)
            affine_map_compose_apply(f1, f2, fcomp, p.first)
            affine_map_compose_apply(g1, g2, gcomp, p.second)
            affine_map_product_apply(fcomp, gcomp, p)
            composed_fun(p) = component_composition.to_fun(p)
        }
        composed_fun = component_composition.to_fun

        AffineMap.new(
            component_composition.src, component_composition.dst,
            component_composition.to_fun) = Option.some(component_composition)
        AffineMap.new(
            first_product.src, second_product.dst, composed_fun) =
            Option.some(component_composition)
        affine_map_compose(first_product, second_product) =
            AffineMap.new(
                first_product.src, second_product.dst, composed_fun)
        affine_map_compose(first_product, second_product) =
            Option.some(component_composition)
        affine_map_compose(
            affine_map_product(f1, g1), affine_map_product(f2, g2)) =
            Option.some(affine_map_product(fcomp, gcomp))
    }
}

/// The first projection commutes with a componentwise product of affine maps.
theorem affine_map_product_fst_natural[V1: AddCommGroup, P1, V2: AddCommGroup, P2, W1: AddCommGroup, Q1, W2: AddCommGroup, Q2](
    f: AffineMap[V1, P1, V2, P2], g: AffineMap[W1, Q1, W2, Q2]
) {
    affine_map_compose(
        affine_map_product(f, g), affine_map_product_fst(f.dst, g.dst)) =
        affine_map_compose(affine_map_product_fst(f.src, g.src), f)
} by {
    let product = affine_map_product(f, g)
    let target_projection = affine_map_product_fst(f.dst, g.dst)
    let source_projection = affine_map_product_fst(f.src, g.src)

    affine_map_product_dst(f, g)
    affine_map_product_fst_src(f.dst, g.dst)
    product.dst = target_projection.src
    affine_map_compose_some(product, target_projection)
    let left: AffineMap[Pair[V1, W1], Pair[P1, Q1], V2, P2] satisfy {
        affine_map_compose(product, target_projection) = Option.some(left)
    }

    affine_map_product_fst_dst(f.src, g.src)
    source_projection.dst = f.src
    affine_map_compose_some(source_projection, f)
    let right: AffineMap[Pair[V1, W1], Pair[P1, Q1], V2, P2] satisfy {
        affine_map_compose(source_projection, f) = Option.some(right)
    }

    affine_map_compose_src(product, target_projection, left)
    affine_map_compose_src(source_projection, f, right)
    affine_map_product_src(f, g)
    affine_map_product_fst_src(f.src, g.src)
    left.src = right.src

    affine_map_compose_dst(product, target_projection, left)
    affine_map_compose_dst(source_projection, f, right)
    affine_map_product_fst_dst(f.dst, g.dst)
    left.dst = right.dst

    forall(p: Pair[P1, Q1]) {
        affine_map_compose_apply(product, target_projection, left, p)
        affine_map_product_apply(f, g, p)
        let middle = product.to_fun(p)
        middle = Pair.new(f.to_fun(p.first), g.to_fun(p.second))
        pair_new_first(f.to_fun(p.first), g.to_fun(p.second))
        affine_map_product_fst_apply(f.dst, g.dst, middle)
        left.to_fun(p) = f.to_fun(p.first)

        affine_map_compose_apply(source_projection, f, right, p)
        affine_map_product_fst_apply(f.src, g.src, p)
        right.to_fun(p) = f.to_fun(p.first)
        left.to_fun(p) = right.to_fun(p)
    }
    affine_map_ext(left, right)
    left = right
    affine_map_compose(product, target_projection) =
        affine_map_compose(source_projection, f)
    affine_map_compose(
        affine_map_product(f, g), affine_map_product_fst(f.dst, g.dst)) =
        affine_map_compose(affine_map_product_fst(f.src, g.src), f)
}

/// The second projection commutes with a componentwise product of affine maps.
theorem affine_map_product_snd_natural[V1: AddCommGroup, P1, V2: AddCommGroup, P2, W1: AddCommGroup, Q1, W2: AddCommGroup, Q2](
    f: AffineMap[V1, P1, V2, P2], g: AffineMap[W1, Q1, W2, Q2]
) {
    affine_map_compose(
        affine_map_product(f, g), affine_map_product_snd(f.dst, g.dst)) =
        affine_map_compose(affine_map_product_snd(f.src, g.src), g)
} by {
    let product = affine_map_product(f, g)
    let target_projection = affine_map_product_snd(f.dst, g.dst)
    let source_projection = affine_map_product_snd(f.src, g.src)

    affine_map_product_dst(f, g)
    affine_map_product_snd_src(f.dst, g.dst)
    product.dst = target_projection.src
    affine_map_compose_some(product, target_projection)
    let left: AffineMap[Pair[V1, W1], Pair[P1, Q1], W2, Q2] satisfy {
        affine_map_compose(product, target_projection) = Option.some(left)
    }

    affine_map_product_snd_dst(f.src, g.src)
    source_projection.dst = g.src
    affine_map_compose_some(source_projection, g)
    let right: AffineMap[Pair[V1, W1], Pair[P1, Q1], W2, Q2] satisfy {
        affine_map_compose(source_projection, g) = Option.some(right)
    }

    affine_map_compose_src(product, target_projection, left)
    affine_map_compose_src(source_projection, g, right)
    affine_map_product_src(f, g)
    affine_map_product_snd_src(f.src, g.src)
    left.src = right.src

    affine_map_compose_dst(product, target_projection, left)
    affine_map_compose_dst(source_projection, g, right)
    affine_map_product_snd_dst(f.dst, g.dst)
    left.dst = right.dst

    forall(p: Pair[P1, Q1]) {
        affine_map_compose_apply(product, target_projection, left, p)
        affine_map_product_apply(f, g, p)
        let middle = product.to_fun(p)
        middle = Pair.new(f.to_fun(p.first), g.to_fun(p.second))
        pair_new_second(f.to_fun(p.first), g.to_fun(p.second))
        affine_map_product_snd_apply(f.dst, g.dst, middle)
        left.to_fun(p) = g.to_fun(p.second)

        affine_map_compose_apply(source_projection, g, right, p)
        affine_map_product_snd_apply(f.src, g.src, p)
        right.to_fun(p) = g.to_fun(p.second)
        left.to_fun(p) = right.to_fun(p)
    }
    affine_map_ext(left, right)
    left = right
    affine_map_compose(product, target_projection) =
        affine_map_compose(source_projection, g)
    affine_map_compose(
        affine_map_product(f, g), affine_map_product_snd(f.dst, g.dst)) =
        affine_map_compose(affine_map_product_snd(f.src, g.src), g)
}
