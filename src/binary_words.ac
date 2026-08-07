from nat import Nat
from combinatorics import binom, choose_n, choose_out_of_bounds, choose_zero, pascal
from data.basic.functions import is_injective_fn
from list import List, add_contains_left, add_contains_right, add_length,
    filter_contained_by_and, filter_contains_and, map, not_contains_add, singleton_unique,
    unique_implies_tail_unique, unique_length, unique_list_sum
from list import is_permutation, permutation_preserves_length,
    unique_same_contains_imp_permutation
from list import injective_map_is_unique, map_contains, map_contains_of_contains,
    map_length

numerals Nat

/// The two kinds of votes in a ballot word.
inductive Vote {
    /// A vote for the leading candidate.
    positive
    /// A vote for the trailing candidate.
    negative
}

/// The number of positive votes in a ballot word.
define positive_count(xs: List[Vote]) -> Nat {
    xs.count(Vote.positive)
}

/// The number of negative votes in a ballot word.
define negative_count(xs: List[Vote]) -> Nat {
    xs.count(Vote.negative)
}

/// True if a ballot word has the prescribed numbers of positive and negative votes.
define ballot_word_has_counts(xs: List[Vote], positives: Nat, negatives: Nat) -> Bool {
    positive_count(xs) = positives and negative_count(xs) = negatives
}

/// Prepending a positive vote increments the positive count.
theorem positive_count_cons_positive(tail: List[Vote]) {
    positive_count(List.cons(Vote.positive, tail)) = Nat.1 + positive_count(tail)
} by {
    List.cons(Vote.positive, tail).count(Vote.positive) = Nat.1 + tail.count(Vote.positive)
}

/// Prepending a positive vote leaves the negative count unchanged.
theorem negative_count_cons_positive(tail: List[Vote]) {
    negative_count(List.cons(Vote.positive, tail)) = negative_count(tail)
} by {
    List.cons(Vote.positive, tail).count(Vote.negative) = tail.count(Vote.negative)
}

/// Prepending a negative vote leaves the positive count unchanged.
theorem positive_count_cons_negative(tail: List[Vote]) {
    positive_count(List.cons(Vote.negative, tail)) = positive_count(tail)
} by {
    List.cons(Vote.negative, tail).count(Vote.positive) = tail.count(Vote.positive)
}

/// Prepending a negative vote increments the negative count.
theorem negative_count_cons_negative(tail: List[Vote]) {
    negative_count(List.cons(Vote.negative, tail)) = Nat.1 + negative_count(tail)
} by {
    List.cons(Vote.negative, tail).count(Vote.negative) = Nat.1 + tail.count(Vote.negative)
}

/// A word made from only positive votes.
define repeat_positive(n: Nat) -> List[Vote] {
    match n {
        Nat.zero {
            List.nil[Vote]
        }
        Nat.suc(k) {
            List.cons(Vote.positive, repeat_positive(k))
        }
    }
}

/// A word made from only negative votes.
define repeat_negative(n: Nat) -> List[Vote] {
    match n {
        Nat.zero {
            List.nil[Vote]
        }
        Nat.suc(k) {
            List.cons(Vote.negative, repeat_negative(k))
        }
    }
}

/// Prepend a positive vote to a ballot word.
define prepend_positive(xs: List[Vote]) -> List[Vote] {
    List.cons(Vote.positive, xs)
}

/// Prepend a negative vote to a ballot word.
define prepend_negative(xs: List[Vote]) -> List[Vote] {
    List.cons(Vote.negative, xs)
}

/// Ballot words of a fixed length with a prescribed number of positive votes.
define ballot_words_len(length: Nat, positives: Nat) -> List[List[Vote]] {
    match length {
        Nat.zero {
            if positives = Nat.0 {
                List.singleton(List.nil[Vote])
            } else {
                List.nil[List[Vote]]
            }
        }
        Nat.suc(tail_length) {
            match positives {
                Nat.zero {
                    map(ballot_words_len(tail_length, Nat.0), prepend_negative)
                }
                Nat.suc(prev_positives) {
                    map(ballot_words_len(tail_length, prev_positives), prepend_positive) +
                        map(ballot_words_len(tail_length, positives), prepend_negative)
                }
            }
        }
    }
}

/// The recursively generated list of ballot words with prescribed vote counts.
define ballot_words(positives: Nat, negatives: Nat) -> List[List[Vote]] {
    ballot_words_len(positives + negatives, positives)
}

/// True if every later nonempty prefix is led by positive votes, from the given prior counts.
define ballot_prefixes_ahead_from(xs: List[Vote], positives_so_far: Nat, negatives_so_far: Nat) -> Bool {
    match xs {
        List.nil[Vote] {
            true
        }
        List.cons(head, tail) {
            match head {
                Vote.positive {
                    negatives_so_far < positives_so_far.suc and
                        ballot_prefixes_ahead_from(tail, positives_so_far.suc, negatives_so_far)
                }
                Vote.negative {
                    negatives_so_far.suc < positives_so_far and
                        ballot_prefixes_ahead_from(tail, positives_so_far, negatives_so_far.suc)
                }
            }
        }
    }
}

/// True if positive votes strictly lead in every nonempty prefix.
define ballot_prefixes_ahead(xs: List[Vote]) -> Bool {
    ballot_prefixes_ahead_from(xs, Nat.0, Nat.0)
}

/// True if every later nonempty prefix has at least as many positive votes as negative votes.
define ballot_prefixes_nonnegative_from(xs: List[Vote], positives_so_far: Nat, negatives_so_far: Nat) -> Bool {
    match xs {
        List.nil[Vote] {
            true
        }
        List.cons(head, tail) {
            match head {
                Vote.positive {
                    negatives_so_far <= positives_so_far.suc and
                        ballot_prefixes_nonnegative_from(tail, positives_so_far.suc, negatives_so_far)
                }
                Vote.negative {
                    negatives_so_far.suc <= positives_so_far and
                        ballot_prefixes_nonnegative_from(tail, positives_so_far, negatives_so_far.suc)
                }
            }
        }
    }
}

/// True if positive votes never fall behind negative votes in any nonempty prefix.
define ballot_prefixes_nonnegative(xs: List[Vote]) -> Bool {
    ballot_prefixes_nonnegative_from(xs, Nat.0, Nat.0)
}

/// A positive weak-prefix step exposes its tail condition.
theorem ballot_prefixes_nonnegative_from_cons_positive_elim(
    tail: List[Vote],
    positives_so_far: Nat,
    negatives_so_far: Nat
) {
    ballot_prefixes_nonnegative_from(List.cons(Vote.positive, tail), positives_so_far, negatives_so_far) implies
        negatives_so_far <= positives_so_far.suc and
        ballot_prefixes_nonnegative_from(tail, positives_so_far.suc, negatives_so_far)
} by {
    if ballot_prefixes_nonnegative_from(List.cons(Vote.positive, tail), positives_so_far, negatives_so_far) {
        ballot_prefixes_nonnegative_from(tail, positives_so_far.suc, negatives_so_far)
    }
}

/// A positive weak-prefix step is built from its tail condition.
theorem ballot_prefixes_nonnegative_from_cons_positive_intro(
    tail: List[Vote],
    positives_so_far: Nat,
    negatives_so_far: Nat
) {
    negatives_so_far <= positives_so_far.suc and
        ballot_prefixes_nonnegative_from(tail, positives_so_far.suc, negatives_so_far)
    implies ballot_prefixes_nonnegative_from(List.cons(Vote.positive, tail), positives_so_far, negatives_so_far)
}

/// A negative weak-prefix step exposes its tail condition.
theorem ballot_prefixes_nonnegative_from_cons_negative_elim(
    tail: List[Vote],
    positives_so_far: Nat,
    negatives_so_far: Nat
) {
    ballot_prefixes_nonnegative_from(List.cons(Vote.negative, tail), positives_so_far, negatives_so_far) implies
        negatives_so_far.suc <= positives_so_far and
        ballot_prefixes_nonnegative_from(tail, positives_so_far, negatives_so_far.suc)
} by {
    if ballot_prefixes_nonnegative_from(List.cons(Vote.negative, tail), positives_so_far, negatives_so_far) {
        ballot_prefixes_nonnegative_from(tail, positives_so_far, negatives_so_far.suc)
    }
}

/// A negative weak-prefix step is built from its tail condition.
theorem ballot_prefixes_nonnegative_from_cons_negative_intro(
    tail: List[Vote],
    positives_so_far: Nat,
    negatives_so_far: Nat
) {
    negatives_so_far.suc <= positives_so_far and
        ballot_prefixes_nonnegative_from(tail, positives_so_far, negatives_so_far.suc)
    implies ballot_prefixes_nonnegative_from(List.cons(Vote.negative, tail), positives_so_far, negatives_so_far)
}

/// True if a ballot word has the requested votes and is strictly led by positive votes.
define successful_ballot_word(xs: List[Vote], positives: Nat, negatives: Nat) -> Bool {
    ballot_word_has_counts(xs, positives, negatives) and ballot_prefixes_ahead(xs)
}

/// The generated list of ballot words where positive votes strictly lead in every nonempty prefix.
define successful_ballot_words(positives: Nat, negatives: Nat) -> List[List[Vote]] {
    ballot_words(positives, negatives).filter(ballot_prefixes_ahead)
}

/// The generated list of ballot words where positive votes never fall behind.
define weak_ballot_words(positives: Nat, negatives: Nat) -> List[List[Vote]] {
    ballot_words(positives, negatives).filter(ballot_prefixes_nonnegative)
}

/// Weak ballot words with an initial positive vote added.
define strict_successful_tails_image(positives: Nat, negatives: Nat) -> List[List[Vote]] {
    map(weak_ballot_words(positives, negatives), prepend_positive)
}

/// The complement of a predicate.
define predicate_not[T](f: T -> Bool, x: T) -> Bool {
    not f(x)
}

/// True if a ballot word has a prefix where negative votes outnumber positive votes.
define ballot_prefixes_negative(xs: List[Vote]) -> Bool {
    not ballot_prefixes_nonnegative(xs)
}

/// Generated ballot words whose prefixes eventually go negative.
define bad_ballot_words(positives: Nat, negatives: Nat) -> List[List[Vote]] {
    ballot_words(positives, negatives).filter(ballot_prefixes_negative)
}

/// In a ballot word, the two vote counts add up to the word length.
theorem ballot_count_sum_length(xs: List[Vote]) {
    positive_count(xs) + negative_count(xs) = xs.length
} by {
    define p(ys: List[Vote]) -> Bool {
        positive_count(ys) + negative_count(ys) = ys.length
    }
    positive_count(List.nil[Vote]) = Nat.0
    negative_count(List.nil[Vote]) = Nat.0
    List.nil[Vote].length = Nat.0
    p(List.nil[Vote])
    forall(head: Vote, tail: List[Vote]) {
        if p(tail) {
            match head {
                Vote.positive {
                    positive_count_cons_positive(tail)
                    negative_count_cons_positive(tail)
                    positive_count(List.cons(head, tail)) + negative_count(List.cons(head, tail)) =
                        Nat.1 + (positive_count(tail) + negative_count(tail))
                    Nat.1 + (positive_count(tail) + negative_count(tail)) = Nat.1 + tail.length
                    Nat.1 + tail.length = tail.length.suc
                    p(List.cons(head, tail))
                }
                Vote.negative {
                    positive_count_cons_negative(tail)
                    negative_count_cons_negative(tail)
                    positive_count(List.cons(head, tail)) + negative_count(List.cons(head, tail)) =
                        Nat.1 + (positive_count(tail) + negative_count(tail))
                    Nat.1 + (positive_count(tail) + negative_count(tail)) = Nat.1 + tail.length
                    Nat.1 + tail.length = tail.length.suc
                    p(List.cons(head, tail))
                }
            }
            p(List.cons(head, tail))
        }
    }
    p(xs)
}

/// Membership in a concatenation comes from one side.
theorem add_contains_cases[T](left: List[T], right: List[T], item: T) {
    (left + right).contains(item) implies left.contains(item) or right.contains(item)
} by {
    if (left + right).contains(item) {
        if left.contains(item) {
            left.contains(item) or right.contains(item)
        }
        if not left.contains(item) {
            if right.contains(item) {
                left.contains(item) or right.contains(item)
            }
            if not right.contains(item) {
                not_contains_add(left, right, item)
                false
            }
        }
    }
}

/// The only element of a singleton is its listed element.
theorem singleton_contains_imp_eq[T](item: T, found: T) {
    List.singleton(item).contains(found) implies found = item
} by {
    if List.singleton(item).contains(found) {
        if found != item {
            not List.singleton(item).contains(found)
            false
        }
    }
}

/// In a unique cons list, the head is absent from the tail.
theorem unique_cons_not_contains[T](head: T, tail: List[T]) {
    List.cons(head, tail).is_unique implies not tail.contains(head)
} by {
    if List.cons(head, tail).is_unique {
        if tail.contains(head) {
            List.cons(head, tail).unique = List.cons(head, tail)
            tail.unique = List.cons(head, tail)
            unique_length(tail)
            List.cons(head, tail).length = tail.length.suc
            tail.length.suc <= tail.length
            false
        }
    }
}

/// Adding a fresh head to a unique tail gives a unique list.
theorem cons_unique_of_tail_unique_not_contains[T](head: T, tail: List[T]) {
    tail.is_unique and not tail.contains(head) implies List.cons(head, tail).is_unique
} by {
    if tail.is_unique and not tail.contains(head) {
        List.cons(head, tail).is_unique
    }
}

/// Filtering a unique list preserves uniqueness.
theorem filter_preserves_unique[T](list: List[T], f: T -> Bool) {
    list.is_unique implies list.filter(f).is_unique
} by {
    define p(xs: List[T]) -> Bool {
        xs.is_unique implies xs.filter(f).is_unique
    }

    p(List.nil[T])
    forall(head: T, tail: List[T]) {
        if p(tail) {
            if List.cons(head, tail).is_unique {
                unique_implies_tail_unique(head, tail)
                unique_cons_not_contains(head, tail)
                if f(head) {
                    if tail.filter(f).contains(head) {
                        filter_contained_by_and(tail, f, head)
                        false
                    }
                    cons_unique_of_tail_unique_not_contains(head, tail.filter(f))
                    List.cons(head, tail).filter(f).is_unique
                }
                if not f(head) {
                    List.cons(head, tail).filter(f).is_unique
                }
                List.cons(head, tail).filter(f).is_unique
            }
            if not List.cons(head, tail).is_unique {
            }
            p(List.cons(head, tail))
        }
    }

}

/// Filtering distributes over list concatenation.
theorem filter_add[T](left: List[T], right: List[T], f: T -> Bool) {
    (left + right).filter(f) = left.filter(f) + right.filter(f)
} by {
    define p(xs: List[T]) -> Bool {
        (xs + right).filter(f) = xs.filter(f) + right.filter(f)
    }

    p(List.nil[T])
    forall(head: T, tail: List[T]) {
        if p(tail) {
            if f(head) {
                List.cons(head, tail).filter(f) = List.cons(head, tail.filter(f))
                (List.cons(head, tail) + right).filter(f) =
                    List.cons(head, tail).filter(f) + right.filter(f)
            }
            if not f(head) {
                (List.cons(head, tail) + right).filter(f) =
                    List.cons(head, tail).filter(f) + right.filter(f)
            }
            p(List.cons(head, tail))
        }
    }

}

/// Prepending a positive vote increments the length.
theorem prepend_positive_length(xs: List[Vote]) {
    prepend_positive(xs).length = xs.length.suc
}

/// Prepending a negative vote increments the length.
theorem prepend_negative_length(xs: List[Vote]) {
    prepend_negative(xs).length = xs.length.suc
}

/// Prepending a positive vote increments the positive count.
theorem prepend_positive_positive_count(xs: List[Vote]) {
    positive_count(prepend_positive(xs)) = positive_count(xs).suc
} by {
    positive_count_cons_positive(xs)
}

/// Prepending a positive vote leaves the negative count unchanged.
theorem prepend_positive_negative_count(xs: List[Vote]) {
    negative_count(prepend_positive(xs)) = negative_count(xs)
} by {
    negative_count_cons_positive(xs)
}

/// Prepending a negative vote leaves the positive count unchanged.
theorem prepend_negative_positive_count(xs: List[Vote]) {
    positive_count(prepend_negative(xs)) = positive_count(xs)
} by {
    positive_count_cons_negative(xs)
}

/// Prepending a negative vote increments the negative count.
theorem prepend_negative_negative_count(xs: List[Vote]) {
    negative_count(prepend_negative(xs)) = negative_count(xs).suc
} by {
    negative_count_cons_negative(xs)
}

/// Prepending a positive vote is injective.
theorem prepend_positive_injective {
    is_injective_fn(prepend_positive)
} by {
    forall(xs: List[Vote], ys: List[Vote]) {
        if prepend_positive(xs) = prepend_positive(ys) {
            List.cons(Vote.positive, xs) = List.cons(Vote.positive, ys)
            xs = ys
        }
    }
}

/// Prepending a negative vote is injective.
theorem prepend_negative_injective {
    is_injective_fn(prepend_negative)
} by {
    forall(xs: List[Vote], ys: List[Vote]) {
        if prepend_negative(xs) = prepend_negative(ys) {
            List.cons(Vote.negative, xs) = List.cons(Vote.negative, ys)
            xs = ys
        }
    }
}

/// A word cannot arise both by prepending a positive vote and by prepending a negative vote.
theorem prepend_positive_negative_disjoint(
    positives: List[List[Vote]],
    negatives: List[List[Vote]],
    word: List[Vote]
) {
    not (
        map(positives, prepend_positive).contains(word) and
            map(negatives, prepend_negative).contains(word)
    )
} by {
    if map(positives, prepend_positive).contains(word) and
        map(negatives, prepend_negative).contains(word) {
        map_contains[List[Vote], List[Vote]](positives, prepend_positive, word)
        let xs: List[Vote] satisfy {
            positives.contains(xs) and prepend_positive(xs) = word
        }
        map_contains[List[Vote], List[Vote]](negatives, prepend_negative, word)
        let ys: List[Vote] satisfy {
            negatives.contains(ys) and prepend_negative(ys) = word
        }
        List.cons(Vote.positive, xs) = List.cons(Vote.negative, ys)
        false
    }
}

/// A word beginning with a positive vote is not in the image of prepending a negative vote.
theorem map_prepend_negative_not_contains_prepend_positive(
    words: List[List[Vote]],
    xs: List[Vote]
) {
    not map(words, prepend_negative).contains(prepend_positive(xs))
} by {
    if map(words, prepend_negative).contains(prepend_positive(xs)) {
        map_contains[List[Vote], List[Vote]](words, prepend_negative, prepend_positive(xs))
        let ys: List[Vote] satisfy {
            words.contains(ys) and prepend_negative(ys) = prepend_positive(xs)
        }
        List.cons(Vote.negative, ys) = List.cons(Vote.positive, xs)
        false
    }
}

/// The empty ballot word vacuously has no bad nonempty prefix.
theorem ballot_prefixes_ahead_nil {
    ballot_prefixes_ahead(List.nil[Vote])
}

/// A ballot word beginning with a negative vote is not strictly led by positive votes.
theorem ballot_prefixes_ahead_cons_negative_false(xs: List[Vote]) {
    not ballot_prefixes_ahead(List.cons(Vote.negative, xs))
} by {
}

/// A positive first vote preserves the prefix condition exactly as expected.
theorem ballot_prefixes_ahead_cons_positive(xs: List[Vote]) {
    ballot_prefixes_ahead_from(xs, Nat.1, Nat.0) implies
        ballot_prefixes_ahead(List.cons(Vote.positive, xs))
} by {
    if ballot_prefixes_ahead_from(xs, Nat.1, Nat.0) {
        Nat.0 < Nat.1
    }
}

/// A strictly led word beginning with a positive vote has a strictly led tail from balance one.
theorem ballot_prefixes_ahead_cons_positive_tail(xs: List[Vote]) {
    ballot_prefixes_ahead(List.cons(Vote.positive, xs)) implies
        ballot_prefixes_ahead_from(xs, Nat.1, Nat.0)
}

/// Strict lead from one extra positive vote implies weak nonnegative balance.
theorem ballot_prefixes_ahead_from_suc_imp_nonnegative_from(
    xs: List[Vote],
    positives_so_far: Nat,
    negatives_so_far: Nat
) {
    ballot_prefixes_ahead_from(xs, positives_so_far.suc, negatives_so_far) implies
        ballot_prefixes_nonnegative_from(xs, positives_so_far, negatives_so_far)
} by {
    define p(ys: List[Vote]) -> Bool {
        forall(a: Nat, b: Nat) {
            ballot_prefixes_ahead_from(ys, a.suc, b) implies
                ballot_prefixes_nonnegative_from(ys, a, b)
        }
    }

    p(List.nil[Vote])
    forall(head: Vote, tail: List[Vote]) {
        if p(tail) {
            forall(a: Nat, b: Nat) {
                match head {
                    Vote.positive {
                        if ballot_prefixes_ahead_from(List.cons(head, tail), a.suc, b) {
                            b < a.suc.suc
                            b <= a.suc
                            ballot_prefixes_nonnegative_from(tail, a.suc, b)
                            ballot_prefixes_nonnegative_from(List.cons(head, tail), a, b)
                        }
                    }
                    Vote.negative {
                        if ballot_prefixes_ahead_from(List.cons(head, tail), a.suc, b) {
                            b.suc < a.suc
                            b.suc <= a
                            ballot_prefixes_nonnegative_from(tail, a, b.suc)
                            ballot_prefixes_nonnegative_from(List.cons(head, tail), a, b)
                        }
                    }
                }
            }
            p(List.cons(head, tail))
        }
    }

    p(xs)
}

/// Weak nonnegative balance implies strict lead from one extra positive vote.
theorem ballot_prefixes_nonnegative_from_imp_ahead_from_suc(
    xs: List[Vote],
    positives_so_far: Nat,
    negatives_so_far: Nat
) {
    ballot_prefixes_nonnegative_from(xs, positives_so_far, negatives_so_far) implies
        ballot_prefixes_ahead_from(xs, positives_so_far.suc, negatives_so_far)
} by {
    define p(ys: List[Vote]) -> Bool {
        forall(a: Nat, b: Nat) {
            ballot_prefixes_nonnegative_from(ys, a, b) implies
                ballot_prefixes_ahead_from(ys, a.suc, b)
        }
    }

    p(List.nil[Vote])
    forall(head: Vote, tail: List[Vote]) {
        if p(tail) {
            forall(a: Nat, b: Nat) {
                match head {
                    Vote.positive {
                        head = Vote.positive
                        if ballot_prefixes_nonnegative_from(List.cons(head, tail), a, b) {
                            b <= a.suc
                            b < a.suc.suc
                            ballot_prefixes_ahead_from(tail, a.suc.suc, b)
                            ballot_prefixes_ahead_from(List.cons(head, tail), a.suc, b)
                        }
                    }
                    Vote.negative {
                        head = Vote.negative
                        if ballot_prefixes_nonnegative_from(List.cons(head, tail), a, b) {
                            b.suc <= a
                            b.suc < a.suc
                            ballot_prefixes_ahead_from(tail, a.suc, b.suc)
                            ballot_prefixes_ahead_from(List.cons(head, tail), a.suc, b)
                        }
                    }
                }
            }
            p(List.cons(head, tail))
        }
    }

    p(xs)
}

/// After an initial positive vote, strict lead implies weak nonnegative balance on the tail.
theorem ballot_prefixes_ahead_cons_positive_imp_nonnegative(xs: List[Vote]) {
    ballot_prefixes_ahead(List.cons(Vote.positive, xs)) implies ballot_prefixes_nonnegative(xs)
} by {
    if ballot_prefixes_ahead(List.cons(Vote.positive, xs)) {
        ballot_prefixes_ahead_cons_positive_tail(xs)
        ballot_prefixes_ahead_from_suc_imp_nonnegative_from(xs, Nat.0, Nat.0)
        ballot_prefixes_nonnegative(xs)
    }
}

/// Weak nonnegative balance on the tail implies strict lead after an initial positive vote.
theorem ballot_prefixes_nonnegative_imp_ahead_cons_positive(xs: List[Vote]) {
    ballot_prefixes_nonnegative(xs) implies ballot_prefixes_ahead(List.cons(Vote.positive, xs))
} by {
    if ballot_prefixes_nonnegative(xs) {
        ballot_prefixes_nonnegative_from_imp_ahead_from_suc(xs, Nat.0, Nat.0)
        ballot_prefixes_ahead_from(xs, Nat.1, Nat.0)
        ballot_prefixes_ahead_cons_positive(xs)
    }
}

/// Generated words with no positive votes have no positive votes and the requested length.
theorem ballot_words_len_zero_sound(length: Nat, word: List[Vote]) {
    ballot_words_len(length, Nat.0).contains(word) implies
        word.length = length and positive_count(word) = Nat.0
} by {
    define p(len: Nat) -> Bool {
        forall(xs: List[Vote]) {
            ballot_words_len(len, Nat.0).contains(xs) implies
                xs.length = len and positive_count(xs) = Nat.0
        }
    }

    forall(xs: List[Vote]) {
        if ballot_words_len(Nat.0, Nat.0).contains(xs) {
            ballot_words_len(Nat.0, Nat.0) = List.singleton(List.nil[Vote])
            singleton_contains_imp_eq[List[Vote]](List.nil[Vote], xs)
            xs = List.nil[Vote]
            positive_count(xs) = Nat.0
            xs.length = Nat.0 and positive_count(xs) = Nat.0
        }
    }
    p(Nat.0)

    forall(len: Nat) {
        if p(len) {
            forall(xs: List[Vote]) {
                if ballot_words_len(len.suc, Nat.0).contains(xs) {
                    ballot_words_len(len.suc, Nat.0) =
                        map(ballot_words_len(len, Nat.0), prepend_negative)
                    map_contains[List[Vote], List[Vote]](
                        ballot_words_len(len, Nat.0),
                        prepend_negative,
                        xs
                    )
                    let ys: List[Vote] satisfy {
                        ballot_words_len(len, Nat.0).contains(ys) and prepend_negative(ys) = xs
                    }
                    prepend_negative_length(ys)
                    ys.length.suc = len.suc
                    prepend_negative_positive_count(ys)
                    positive_count(xs) = positive_count(ys)
                    positive_count(xs) = Nat.0
                    xs.length = len.suc and positive_count(xs) = Nat.0
                }
            }
            p(len.suc)
        }
    }

    p(length)
}

/// Every generated fixed-length word has the requested length and positive count.
theorem ballot_words_len_sound(length: Nat, positives: Nat, word: List[Vote]) {
    ballot_words_len(length, positives).contains(word) implies
        word.length = length and positive_count(word) = positives
} by {
    define p(len: Nat) -> Bool {
        forall(k: Nat, xs: List[Vote]) {
            ballot_words_len(len, k).contains(xs) implies
                xs.length = len and positive_count(xs) = k
        }
    }

    forall(k: Nat, xs: List[Vote]) {
        if ballot_words_len(Nat.0, k).contains(xs) {
            if k = Nat.0 {
                ballot_words_len(Nat.0, k) = List.singleton(List.nil[Vote])
                singleton_contains_imp_eq[List[Vote]](List.nil[Vote], xs)
                xs = List.nil[Vote]
                xs.length = Nat.0
                positive_count(xs) = Nat.0
                xs.length = Nat.0 and positive_count(xs) = k
            }
            if k != Nat.0 {
                ballot_words_len(Nat.0, k) = List.nil[List[Vote]]
                false
            }
            ballot_words_len(Nat.0, k).contains(xs) implies xs.length = Nat.0 and positive_count(xs) = k
        }
    }
    p(Nat.0) = forall(k: Nat, xs: List[Vote]) {
        ballot_words_len(Nat.0, k).contains(xs) implies
            xs.length = Nat.0 and positive_count(xs) = k
    }
    if not p(Nat.0) {
        let (k0: Nat, xs0: List[Vote]) satisfy {
            ballot_words_len(Nat.0, k0).contains(xs0) and not (xs0.length = Nat.0 and positive_count(xs0) = k0)
        }
        if k0 = Nat.0 {
            singleton_contains_imp_eq[List[Vote]](List.nil[Vote], xs0)
            false
        } else {
            false
        }
    }
    p(Nat.0)

    forall(len: Nat) {
        if p(len) {
            forall(k: Nat, xs: List[Vote]) {
                if ballot_words_len(len.suc, k).contains(xs) {
                    match k {
                        Nat.zero {
                            ballot_words_len_zero_sound(len.suc, xs)
                            xs.length = len.suc
                            positive_count(xs) = Nat.0
                            xs.length = len.suc and positive_count(xs) = k
                        }
                        Nat.suc(prev) {
                            ballot_words_len(len.suc, k) =
                                map(ballot_words_len(len, prev), prepend_positive) +
                                    map(ballot_words_len(len, k), prepend_negative)
                            add_contains_cases[List[Vote]](
                                map(ballot_words_len(len, prev), prepend_positive),
                                map(ballot_words_len(len, k), prepend_negative),
                                xs
                            )
                            if map(ballot_words_len(len, prev), prepend_positive).contains(xs) {
                                map_contains[List[Vote], List[Vote]](
                                    ballot_words_len(len, prev),
                                    prepend_positive,
                                    xs
                                )
                                let ys: List[Vote] satisfy {
                                    ballot_words_len(len, prev).contains(ys) and prepend_positive(ys) = xs
                                }
                                p(len)
                                positive_count(ys) = prev
                                prepend_positive_length(ys)
                                ys.length.suc = len.suc
                                prepend_positive_positive_count(ys)
                                positive_count(xs) = k
                                xs.length = len.suc and positive_count(xs) = k
                            }
                            if map(ballot_words_len(len, k), prepend_negative).contains(xs) {
                                map_contains[List[Vote], List[Vote]](
                                    ballot_words_len(len, k),
                                    prepend_negative,
                                    xs
                                )
                                let ys: List[Vote] satisfy {
                                    ballot_words_len(len, k).contains(ys) and prepend_negative(ys) = xs
                                }
                                p(len)
                                prepend_negative_length(ys)
                                ys.length.suc = len.suc
                                prepend_negative_positive_count(ys)
                                positive_count(xs) = positive_count(ys)
                                positive_count(xs) = k
                                xs.length = len.suc and positive_count(xs) = k
                            }
                            if not (xs.length = len.suc and positive_count(xs) = k) {
                                if map(ballot_words_len(len, prev), prepend_positive).contains(xs) {
                                    false
                                }
                                if map(ballot_words_len(len, k), prepend_negative).contains(xs) {
                                    false
                                }
                                false
                            }
                            xs.length = len.suc and positive_count(xs) = k
                        }
                    }
                    xs.length = len.suc and positive_count(xs) = k
                }
            }
            p(len.suc)
        }
    }

    p(length)
}

/// Every generated ballot word has the requested positive and negative counts.
theorem ballot_words_sound(positives: Nat, negatives: Nat, word: List[Vote]) {
    ballot_words(positives, negatives).contains(word) implies
        ballot_word_has_counts(word, positives, negatives)
} by {
    if ballot_words(positives, negatives).contains(word) {
        ballot_words_len_sound(positives + negatives, positives, word)
        word.length = positives + negatives
        positive_count(word) = positives
        ballot_count_sum_length(word)
        positives + negative_count(word) = positives + negatives
        ballot_word_has_counts(word, positives, negatives)
    }
}

/// Every word with the requested length and positive count is generated.
theorem ballot_words_len_complete(length: Nat, positives: Nat, word: List[Vote]) {
    word.length = length and positive_count(word) = positives implies
        ballot_words_len(length, positives).contains(word)
} by {
    define p(xs: List[Vote]) -> Bool {
        forall(len: Nat, k: Nat) {
            xs.length = len and positive_count(xs) = k implies
                ballot_words_len(len, k).contains(xs)
        }
    }

    forall(len: Nat, k: Nat) {
        if List.nil[Vote].length = len and positive_count(List.nil[Vote]) = k {
            List.nil[Vote].length = Nat.0
            positive_count(List.nil[Vote]) = Nat.0
            len = Nat.0
            k = Nat.0
            ballot_words_len(len, k) = List.singleton(List.nil[Vote])
            ballot_words_len(len, k).contains(List.nil[Vote])
        }
    }
    p(List.nil[Vote])

    forall(head: Vote, tail: List[Vote]) {
        if p(tail) {
            forall(len: Nat, k: Nat) {
                if List.cons(head, tail).length = len and positive_count(List.cons(head, tail)) = k {
                    match head {
                        Vote.positive {
                            positive_count_cons_positive(tail)
                            map_contains_of_contains[List[Vote], List[Vote]](
                                ballot_words_len(tail.length, positive_count(tail)),
                                prepend_positive,
                                tail
                            )
                            map(ballot_words_len(tail.length, positive_count(tail)), prepend_positive).contains(
                                prepend_positive(tail)
                            )
                            ballot_words_len(len, k) =
                                map(ballot_words_len(tail.length, positive_count(tail)), prepend_positive) +
                                    map(ballot_words_len(tail.length, positive_count(tail).suc), prepend_negative)
                            add_contains_left[List[Vote]](
                                map(ballot_words_len(tail.length, positive_count(tail)), prepend_positive),
                                map(ballot_words_len(tail.length, positive_count(tail).suc), prepend_negative),
                                List.cons(head, tail)
                            )
                            (
                                map(ballot_words_len(tail.length, positive_count(tail)), prepend_positive) +
                                    map(ballot_words_len(tail.length, positive_count(tail).suc), prepend_negative)
                            ).contains(List.cons(head, tail))
                            ballot_words_len(len, k).contains(List.cons(head, tail))
                        }
                        Vote.negative {
                            positive_count_cons_negative(tail)
                            map_contains_of_contains[List[Vote], List[Vote]](
                                ballot_words_len(tail.length, positive_count(tail)),
                                prepend_negative,
                                tail
                            )
                            map(ballot_words_len(tail.length, positive_count(tail)), prepend_negative).contains(
                                prepend_negative(tail)
                            )
                            match positive_count(tail) {
                                Nat.zero {
                                    k = Nat.0
                                    ballot_words_len(len, k) =
                                        map(ballot_words_len(tail.length, Nat.0), prepend_negative)
                                    ballot_words_len(len, k).contains(List.cons(head, tail))
                                }
                                Nat.suc(prev) {
                                    k = prev.suc
                                    ballot_words_len(len, k) =
                                        map(ballot_words_len(tail.length, prev), prepend_positive) +
                                            map(ballot_words_len(tail.length, k), prepend_negative)
                                    add_contains_right[List[Vote]](
                                        map(ballot_words_len(tail.length, prev), prepend_positive),
                                        map(ballot_words_len(tail.length, k), prepend_negative),
                                        List.cons(head, tail)
                                    )
                                    (
                                        map(ballot_words_len(tail.length, prev), prepend_positive) +
                                            map(ballot_words_len(tail.length, k), prepend_negative)
                                    ).contains(List.cons(head, tail))
                                    ballot_words_len(len, k).contains(List.cons(head, tail))
                                }
                            }
                        }
                    }
                    ballot_words_len(len, k).contains(List.cons(head, tail))
                }
            }
            p(List.cons(head, tail))
        }
    }

    p(word)
}

/// Every word with the requested positive and negative counts is generated.
theorem ballot_words_complete(positives: Nat, negatives: Nat, word: List[Vote]) {
    ballot_word_has_counts(word, positives, negatives) implies
        ballot_words(positives, negatives).contains(word)
} by {
    if ballot_word_has_counts(word, positives, negatives) {
        positive_count(word) = positives
        negative_count(word) = negatives
        ballot_count_sum_length(word)
        word.length = positives + negatives
        ballot_words_len_complete(positives + negatives, positives, word)
        ballot_words(positives, negatives).contains(word)
    }
}

/// Every generated successful ballot word has the requested counts and strict prefix lead.
theorem successful_ballot_words_sound(positives: Nat, negatives: Nat, word: List[Vote]) {
    successful_ballot_words(positives, negatives).contains(word) implies
        successful_ballot_word(word, positives, negatives)
} by {
    if successful_ballot_words(positives, negatives).contains(word) {
        filter_contained_by_and[List[Vote]](
            ballot_words(positives, negatives),
            ballot_prefixes_ahead,
            word
        )
        ballot_words(positives, negatives).contains(word)
        ballot_prefixes_ahead(word)
        ballot_words_sound(positives, negatives, word)
        successful_ballot_word(word, positives, negatives)
    }
}

/// True if a ballot word has the requested votes and never falls behind.
define weak_ballot_word(xs: List[Vote], positives: Nat, negatives: Nat) -> Bool {
    ballot_word_has_counts(xs, positives, negatives) and ballot_prefixes_nonnegative(xs)
}

/// Every generated weak ballot word has the requested counts and weak prefix property.
theorem weak_ballot_words_sound(positives: Nat, negatives: Nat, word: List[Vote]) {
    weak_ballot_words(positives, negatives).contains(word) implies
        weak_ballot_word(word, positives, negatives)
} by {
    if weak_ballot_words(positives, negatives).contains(word) {
        filter_contained_by_and[List[Vote]](
            ballot_words(positives, negatives),
            ballot_prefixes_nonnegative,
            word
        )
        ballot_words(positives, negatives).contains(word)
        ballot_prefixes_nonnegative(word)
        ballot_words_sound(positives, negatives, word)
        weak_ballot_word(word, positives, negatives)
    }
}

/// Every word satisfying the strict ballot predicate is generated by the strict ballot list.
theorem successful_ballot_words_complete(positives: Nat, negatives: Nat, word: List[Vote]) {
    successful_ballot_word(word, positives, negatives) implies
        successful_ballot_words(positives, negatives).contains(word)
} by {
    if successful_ballot_word(word, positives, negatives) {
        ballot_word_has_counts(word, positives, negatives)
        ballot_prefixes_ahead(word)
        ballot_words_complete(positives, negatives, word)
        filter_contains_and[List[Vote]](
            ballot_words(positives, negatives),
            ballot_prefixes_ahead,
            word
        )
        successful_ballot_words(positives, negatives).contains(word)
    }
}

/// Membership in the strict ballot list is exactly the strict ballot predicate.
theorem successful_ballot_words_contains_iff(positives: Nat, negatives: Nat, word: List[Vote]) {
    successful_ballot_words(positives, negatives).contains(word) =
        successful_ballot_word(word, positives, negatives)
} by {
    if successful_ballot_words(positives, negatives).contains(word) {
        successful_ballot_words_sound(positives, negatives, word)
        successful_ballot_word(word, positives, negatives)
    }
    if successful_ballot_word(word, positives, negatives) {
        successful_ballot_words_complete(positives, negatives, word)
        successful_ballot_words(positives, negatives).contains(word)
    }
}

/// Every word satisfying the weak ballot predicate is generated by the weak ballot list.
theorem weak_ballot_words_complete(positives: Nat, negatives: Nat, word: List[Vote]) {
    weak_ballot_word(word, positives, negatives) implies
        weak_ballot_words(positives, negatives).contains(word)
} by {
    if weak_ballot_word(word, positives, negatives) {
        ballot_word_has_counts(word, positives, negatives)
        ballot_prefixes_nonnegative(word)
        ballot_words_complete(positives, negatives, word)
        filter_contains_and[List[Vote]](
            ballot_words(positives, negatives),
            ballot_prefixes_nonnegative,
            word
        )
        weak_ballot_words(positives, negatives).contains(word)
    }
}

/// Membership in the weak ballot list is exactly the weak ballot predicate.
theorem weak_ballot_words_contains_iff(positives: Nat, negatives: Nat, word: List[Vote]) {
    weak_ballot_words(positives, negatives).contains(word) = weak_ballot_word(word, positives, negatives)
} by {
    if weak_ballot_words(positives, negatives).contains(word) {
        weak_ballot_words_sound(positives, negatives, word)
        weak_ballot_word(word, positives, negatives)
    }
    if weak_ballot_word(word, positives, negatives) {
        weak_ballot_words_complete(positives, negatives, word)
        weak_ballot_words(positives, negatives).contains(word)
    }
}

/// Every generated bad ballot word has the requested counts and a negative prefix.
theorem bad_ballot_words_sound(positives: Nat, negatives: Nat, word: List[Vote]) {
    bad_ballot_words(positives, negatives).contains(word) implies
        ballot_word_has_counts(word, positives, negatives) and ballot_prefixes_negative(word)
} by {
    if bad_ballot_words(positives, negatives).contains(word) {
        filter_contained_by_and[List[Vote]](
            ballot_words(positives, negatives),
            ballot_prefixes_negative,
            word
        )
        ballot_words(positives, negatives).contains(word)
        ballot_words_sound(positives, negatives, word)
        ballot_word_has_counts(word, positives, negatives)
        ballot_word_has_counts(word, positives, negatives) and ballot_prefixes_negative(word)
    }
}

/// Every word with the requested counts and a negative prefix is generated by the bad ballot list.
theorem bad_ballot_words_complete(positives: Nat, negatives: Nat, word: List[Vote]) {
    ballot_word_has_counts(word, positives, negatives) and ballot_prefixes_negative(word) implies
        bad_ballot_words(positives, negatives).contains(word)
} by {
    if ballot_word_has_counts(word, positives, negatives) and ballot_prefixes_negative(word) {
        ballot_words_complete(positives, negatives, word)
        filter_contains_and[List[Vote]](
            ballot_words(positives, negatives),
            ballot_prefixes_negative,
            word
        )
        bad_ballot_words(positives, negatives).contains(word)
    }
}

/// Membership in the bad ballot list is exactly counts plus a negative prefix.
theorem bad_ballot_words_contains_iff(positives: Nat, negatives: Nat, word: List[Vote]) {
    bad_ballot_words(positives, negatives).contains(word) =
        (ballot_word_has_counts(word, positives, negatives) and ballot_prefixes_negative(word))
} by {
    if bad_ballot_words(positives, negatives).contains(word) {
        bad_ballot_words_sound(positives, negatives, word)
        ballot_word_has_counts(word, positives, negatives) and ballot_prefixes_negative(word)
    }
    if ballot_word_has_counts(word, positives, negatives) and ballot_prefixes_negative(word) {
        bad_ballot_words_complete(positives, negatives, word)
        bad_ballot_words(positives, negatives).contains(word)
    }
}

/// Prepending one positive vote turns a weak ballot word into a strictly led word.
theorem weak_ballot_word_imp_successful_prepend_positive(
    xs: List[Vote],
    positives: Nat,
    negatives: Nat
) {
    weak_ballot_word(xs, positives, negatives) implies
        successful_ballot_word(prepend_positive(xs), positives.suc, negatives)
} by {
    if weak_ballot_word(xs, positives, negatives) {
        ballot_word_has_counts(xs, positives, negatives)
        positive_count(xs) = positives
        prepend_positive_positive_count(xs)
        prepend_positive_negative_count(xs)
        negative_count(prepend_positive(xs)) = negatives
        ballot_word_has_counts(prepend_positive(xs), positives.suc, negatives)
        ballot_prefixes_nonnegative(xs)
        ballot_prefixes_nonnegative_imp_ahead_cons_positive(xs)
        successful_ballot_word(prepend_positive(xs), positives.suc, negatives)
    }
}

/// Removing the initial positive vote from a strictly led word gives a weak ballot word.
theorem successful_prepend_positive_imp_weak_ballot_word(
    xs: List[Vote],
    positives: Nat,
    negatives: Nat
) {
    successful_ballot_word(prepend_positive(xs), positives.suc, negatives) implies
        weak_ballot_word(xs, positives, negatives)
} by {
    if successful_ballot_word(prepend_positive(xs), positives.suc, negatives) {
        ballot_word_has_counts(prepend_positive(xs), positives.suc, negatives)
        prepend_positive_positive_count(xs)
        prepend_positive_negative_count(xs)
        positive_count(xs).suc = positives.suc
        negative_count(xs) = negatives
        ballot_word_has_counts(xs, positives, negatives)
        ballot_prefixes_ahead(prepend_positive(xs))
        ballot_prefixes_ahead_cons_positive_imp_nonnegative(xs)
        ballot_prefixes_nonnegative(xs)
        weak_ballot_word(xs, positives, negatives)
    }
}

/// Membership in the weak-tail list maps into membership in the strict-success list.
theorem weak_ballot_words_contains_imp_successful_prepend_contains(
    positives: Nat,
    negatives: Nat,
    xs: List[Vote]
) {
    weak_ballot_words(positives, negatives).contains(xs) implies
        successful_ballot_words(positives.suc, negatives).contains(prepend_positive(xs))
} by {
    if weak_ballot_words(positives, negatives).contains(xs) {
        filter_contained_by_and[List[Vote]](
            ballot_words(positives, negatives),
            ballot_prefixes_nonnegative,
            xs
        )
        ballot_words(positives, negatives).contains(xs)
        ballot_prefixes_nonnegative(xs)

        map_contains_of_contains[List[Vote], List[Vote]](
            ballot_words_len(positives + negatives, positives),
            prepend_positive,
            xs
        )
        map(ballot_words_len(positives + negatives, positives), prepend_positive).contains(prepend_positive(xs))

        add_contains_left[List[Vote]](
            map(ballot_words_len(positives + negatives, positives), prepend_positive),
            map(ballot_words_len(positives + negatives, positives.suc), prepend_negative),
            prepend_positive(xs)
        )
        ballot_words(positives.suc, negatives).contains(prepend_positive(xs))

        ballot_prefixes_nonnegative_imp_ahead_cons_positive(xs)
        filter_contains_and[List[Vote]](
            ballot_words(positives.suc, negatives),
            ballot_prefixes_ahead,
            prepend_positive(xs)
        )
        successful_ballot_words(positives.suc, negatives).contains(prepend_positive(xs))
    }
}

/// Membership in the strict-success list with an initial positive vote comes from a weak tail.
theorem successful_prepend_contains_imp_weak_ballot_words_contains(
    positives: Nat,
    negatives: Nat,
    xs: List[Vote]
) {
    successful_ballot_words(positives.suc, negatives).contains(prepend_positive(xs)) implies
        weak_ballot_words(positives, negatives).contains(xs)
} by {
    if successful_ballot_words(positives.suc, negatives).contains(prepend_positive(xs)) {
        successful_ballot_words(positives.suc, negatives) =
            ballot_words(positives.suc, negatives).filter(ballot_prefixes_ahead)
        filter_contained_by_and[List[Vote]](
            ballot_words(positives.suc, negatives),
            ballot_prefixes_ahead,
            prepend_positive(xs)
        )
        ballot_words(positives.suc, negatives).contains(prepend_positive(xs))
        ballot_prefixes_ahead(prepend_positive(xs))

        positives.suc + negatives = (positives + negatives).suc
        ballot_words(positives.suc, negatives) =
            ballot_words_len((positives + negatives).suc, positives.suc)
        ballot_words_len((positives + negatives).suc, positives.suc) =
            map(ballot_words_len(positives + negatives, positives), prepend_positive) +
                map(ballot_words_len(positives + negatives, positives.suc), prepend_negative)
        add_contains_cases[List[Vote]](
            map(ballot_words_len(positives + negatives, positives), prepend_positive),
            map(ballot_words_len(positives + negatives, positives.suc), prepend_negative),
            prepend_positive(xs)
        )
        if map(ballot_words_len(positives + negatives, positives), prepend_positive).contains(prepend_positive(xs)) {
            map_contains[List[Vote], List[Vote]](
                ballot_words_len(positives + negatives, positives),
                prepend_positive,
                prepend_positive(xs)
            )
            let ys: List[Vote] satisfy {
                ballot_words_len(positives + negatives, positives).contains(ys) and
                    prepend_positive(ys) = prepend_positive(xs)
            }
            prepend_positive_injective
            is_injective_fn(prepend_positive)
            ys = xs
            ballot_words_len(positives + negatives, positives).contains(xs)
        }
        if map(ballot_words_len(positives + negatives, positives.suc), prepend_negative).contains(prepend_positive(xs)) {
            map_prepend_negative_not_contains_prepend_positive(
                ballot_words_len(positives + negatives, positives.suc),
                xs
            )
            false
        }
        ballot_words_len(positives + negatives, positives).contains(xs)
        ballot_words(positives, negatives).contains(xs)

        prepend_positive(xs) = List.cons(Vote.positive, xs)
        ballot_prefixes_ahead_cons_positive_imp_nonnegative(xs)
        ballot_prefixes_nonnegative(xs)
        filter_contains_and[List[Vote]](
            ballot_words(positives, negatives),
            ballot_prefixes_nonnegative,
            xs
        )
        weak_ballot_words(positives, negatives).contains(xs)
    }
}

/// The prepend-positive image of weak ballot words has exactly the successful ballot words.
theorem strict_successful_tails_image_same_contains(
    positives: Nat,
    negatives: Nat,
    word: List[Vote]
) {
    strict_successful_tails_image(positives, negatives).contains(word) =
        successful_ballot_words(positives.suc, negatives).contains(word)
} by {
    if strict_successful_tails_image(positives, negatives).contains(word) {
        map_contains[List[Vote], List[Vote]](
            weak_ballot_words(positives, negatives),
            prepend_positive,
            word
        )
        let xs: List[Vote] satisfy {
            weak_ballot_words(positives, negatives).contains(xs) and prepend_positive(xs) = word
        }
        weak_ballot_words_contains_imp_successful_prepend_contains(positives, negatives, xs)
        successful_ballot_words(positives.suc, negatives).contains(word)
    }

    if successful_ballot_words(positives.suc, negatives).contains(word) {
        successful_ballot_words_sound(positives.suc, negatives, word)
        ballot_word_has_counts(word, positives.suc, negatives)
        positive_count(word) = positives.suc
        ballot_prefixes_ahead(word)
        match word {
            List.nil[Vote] {
                positive_count(word) = Nat.0
                false
            }
            List.cons(head, tail) {
                match head {
                    Vote.positive {
                        prepend_positive(tail) = word
                        successful_prepend_contains_imp_weak_ballot_words_contains(positives, negatives, tail)
                        map_contains_of_contains[List[Vote], List[Vote]](
                            weak_ballot_words(positives, negatives),
                            prepend_positive,
                            tail
                        )
                        map(weak_ballot_words(positives, negatives), prepend_positive).contains(prepend_positive(tail))
                        strict_successful_tails_image(positives, negatives).contains(word)
                    }
                    Vote.negative {
                        ballot_prefixes_ahead_cons_negative_false(tail)
                        false
                    }
                }
            }
        }
    }
}

/// There is one generated ballot word of length zero with no positive votes.
theorem ballot_words_len_zero_zero_length {
    ballot_words_len(Nat.0, Nat.0).length = Nat.1
} by {
    ballot_words_len(Nat.0, Nat.0) = List.singleton(List.nil[Vote])
    List.singleton(List.nil[Vote]) = List.cons(List.nil[Vote], List.nil[List[Vote]])
    List.nil[List[Vote]].length = Nat.0
    List.cons(List.nil[Vote], List.nil[List[Vote]]).length = Nat.1
}

/// There are no generated ballot words of length zero with a positive vote.
theorem ballot_words_len_zero_suc_length(k: Nat) {
    ballot_words_len(Nat.0, k.suc).length = Nat.0
} by {
    ballot_words_len(Nat.0, k.suc) = List.nil[List[Vote]]
}

/// Extending zero-positive words by a negative vote preserves their count.
theorem ballot_words_len_suc_zero_length(len: Nat) {
    ballot_words_len(len.suc, Nat.0).length = ballot_words_len(len, Nat.0).length
} by {
    map_length[List[Vote], List[Vote]](ballot_words_len(len, Nat.0), prepend_negative)
}

/// The generated ballot word counts satisfy Pascal's recurrence.
theorem ballot_words_len_suc_suc_length(len: Nat, k: Nat) {
    ballot_words_len(len.suc, k.suc).length =
        ballot_words_len(len, k).length + ballot_words_len(len, k.suc).length
} by {
    let left: List[List[Vote]] = map(ballot_words_len(len, k), prepend_positive)
    let right: List[List[Vote]] = map(ballot_words_len(len, k.suc), prepend_negative)
    ballot_words_len(len.suc, k.suc) = left + right
    add_length[List[Vote]](left, right)
    map_length[List[Vote], List[Vote]](ballot_words_len(len, k), prepend_positive)
    map_length[List[Vote], List[Vote]](ballot_words_len(len, k.suc), prepend_negative)
}

/// The zero-length count agrees with the corresponding binomial coefficient.
theorem ballot_words_len_zero_binom(k: Nat) {
    ballot_words_len(Nat.0, k).length = Nat.0.binom(k)
} by {
    match k {
        Nat.zero {
            ballot_words_len_zero_zero_length
            choose_zero(Nat.0)
            ballot_words_len(Nat.0, k).length = Nat.0.binom(k)
        }
        Nat.suc(prev) {
            ballot_words_len_zero_suc_length(prev)
            Nat.0 < k
            choose_out_of_bounds(Nat.0, k)
            ballot_words_len(Nat.0, k).length = Nat.0.binom(k)
        }
    }
}

/// The zero-positive generated count agrees with the corresponding binomial coefficient.
theorem ballot_words_len_zero_positive_binom(length: Nat) {
    ballot_words_len(length, Nat.0).length = length.binom(Nat.0)
} by {
    define p(len: Nat) -> Bool {
        ballot_words_len(len, Nat.0).length = len.binom(Nat.0)
    }
    ballot_words_len_zero_zero_length
    choose_zero(Nat.0)
    p(Nat.0)
    forall(len: Nat) {
        if p(len) {
            ballot_words_len_suc_zero_length(len)
            ballot_words_len(len, Nat.0).length = len.binom(Nat.0)
            choose_zero(len)
            choose_zero(len.suc)
            p(len.suc)
        }
    }
    p(length)
}

/// The generated fixed-length ballot words are counted by binomial coefficients.
theorem ballot_words_len_length_binom(length: Nat, positives: Nat) {
    ballot_words_len(length, positives).length = length.binom(positives)
} by {
    define p(len: Nat) -> Bool {
        forall(k: Nat) {
            ballot_words_len(len, k).length = len.binom(k)
        }
    }

    forall(k: Nat) {
        ballot_words_len_zero_binom(k)
    }
    p(Nat.0)

    forall(len: Nat) {
        if p(len) {
            forall(k: Nat) {
                match k {
                    Nat.zero {
                        ballot_words_len_zero_positive_binom(len.suc)
                        ballot_words_len(len.suc, k).length = len.suc.binom(k)
                    }
                    Nat.suc(prev) {
                        ballot_words_len_suc_suc_length(len, prev)
                        ballot_words_len(len.suc, k).length =
                            ballot_words_len(len, prev).length + ballot_words_len(len, k).length
                        ballot_words_len(len, prev).length = len.binom(prev)
                        ballot_words_len(len, k).length = len.binom(k)
                        ballot_words_len(len.suc, k).length = len.binom(prev) + len.binom(k)
                        if k <= len {
                            Nat.0 < k
                            prev = k - Nat.1
                            pascal(len, k)
                            ballot_words_len(len.suc, k).length = len.suc.binom(k)
                        }
                        if not k <= len {
                            len < k
                            if k = len.suc {
                                choose_n(len)
                                choose_n(len.suc)
                                choose_out_of_bounds(len, k)
                                ballot_words_len(len.suc, k).length = len.suc.binom(k)
                            }
                            if k != len.suc {
                                len.suc < k
                                len < prev
                                choose_out_of_bounds(len.suc, k)
                                choose_out_of_bounds(len, prev)
                                choose_out_of_bounds(len, k)
                                ballot_words_len(len.suc, k).length = len.suc.binom(k)
                            }
                        }
                        if not ballot_words_len(len.suc, k).length = len.suc.binom(k) {
                            if k <= len {
                                false
                            }
                            if not k <= len {
                                if k = len.suc {
                                    prev = len
                                    choose_n(len)
                                    choose_n(len.suc)
                                    choose_out_of_bounds(len, k)
                                    len.binom(prev) = Nat.1
                                    len.binom(k) = Nat.0
                                    false
                                }
                                if k != len.suc {
                                    choose_out_of_bounds(len.suc, k)
                                    choose_out_of_bounds(len, prev)
                                    choose_out_of_bounds(len, k)
                                    false
                                }
                                false
                            }
                        }
                        ballot_words_len(len.suc, k).length = len.suc.binom(k)
                    }
                }
                ballot_words_len(len.suc, k).length = len.suc.binom(k)
            }
            p(len.suc)
        }
    }

    p(length)
}

/// The generated ballot words with prescribed vote counts are counted by a binomial coefficient.
theorem ballot_words_length_binom(positives: Nat, negatives: Nat) {
    ballot_words(positives, negatives).length = (positives + negatives).binom(positives)
} by {
    ballot_words_len_length_binom(positives + negatives, positives)
}

/// The generated fixed-length ballot word list has no duplicate words.
theorem ballot_words_len_unique(length: Nat, positives: Nat) {
    ballot_words_len(length, positives).is_unique
} by {
    define p(len: Nat) -> Bool {
        forall(k: Nat) {
            ballot_words_len(len, k).is_unique
        }
    }

    forall(k: Nat) {
        match k {
            Nat.zero {
                ballot_words_len(Nat.0, k) = List.singleton(List.nil[Vote])
                singleton_unique[List[Vote]](List.nil[Vote])
                ballot_words_len(Nat.0, k).is_unique
            }
            Nat.suc(prev) {
                ballot_words_len(Nat.0, k) = List.nil[List[Vote]]
                ballot_words_len(Nat.0, k).is_unique
            }
        }
    }
    p(Nat.0)

    forall(len: Nat) {
        if p(len) {
            forall(k: Nat) {
                match k {
                    Nat.zero {
                        ballot_words_len(len.suc, k) = map(ballot_words_len(len, Nat.0), prepend_negative)
                        ballot_words_len(len, Nat.0).is_unique
                        prepend_negative_injective
                        injective_map_is_unique[List[Vote], List[Vote]](
                            ballot_words_len(len, Nat.0),
                            prepend_negative
                        )
                        ballot_words_len(len.suc, k).is_unique
                    }
                    Nat.suc(prev) {
                        let left: List[List[Vote]] = map(ballot_words_len(len, prev), prepend_positive)
                        let right: List[List[Vote]] = map(ballot_words_len(len, k), prepend_negative)
                        ballot_words_len(len.suc, k) =
                            map(ballot_words_len(len, prev), prepend_positive) +
                                map(ballot_words_len(len, k), prepend_negative)
                        prepend_positive_injective
                        prepend_negative_injective
                        injective_map_is_unique[List[Vote], List[Vote]](
                            ballot_words_len(len, prev),
                            prepend_positive
                        )
                        injective_map_is_unique[List[Vote], List[Vote]](
                            ballot_words_len(len, k),
                            prepend_negative
                        )
                        left.is_unique
                        right.is_unique
                        forall(word: List[Vote]) {
                            prepend_positive_negative_disjoint(
                                ballot_words_len(len, prev),
                                ballot_words_len(len, k),
                                word
                            )
                            not (left.contains(word) and right.contains(word))
                        }
                        unique_list_sum[List[Vote]](left, right)
                        (left + right).is_unique
                        ballot_words_len(len.suc, k).is_unique
                    }
                }
            }
            p(len.suc)
        }
    }

    p(length)
}

/// The generated ballot words with prescribed vote counts have no duplicate words.
theorem ballot_words_unique(positives: Nat, negatives: Nat) {
    ballot_words(positives, negatives).is_unique
} by {
    ballot_words_len_unique(positives + negatives, positives)
}

/// The successful ballot words with prescribed vote counts have no duplicate words.
theorem successful_ballot_words_unique(positives: Nat, negatives: Nat) {
    successful_ballot_words(positives, negatives).is_unique
} by {
    ballot_words_unique(positives, negatives)
    filter_preserves_unique(ballot_words(positives, negatives), ballot_prefixes_ahead)
}

/// The weak ballot words with prescribed vote counts have no duplicate words.
theorem weak_ballot_words_unique(positives: Nat, negatives: Nat) {
    weak_ballot_words(positives, negatives).is_unique
} by {
    ballot_words_unique(positives, negatives)
    filter_preserves_unique(ballot_words(positives, negatives), ballot_prefixes_nonnegative)
}

/// The bad ballot words with prescribed vote counts have no duplicate words.
theorem bad_ballot_words_unique(positives: Nat, negatives: Nat) {
    bad_ballot_words(positives, negatives).is_unique
} by {
    ballot_words_unique(positives, negatives)
    filter_preserves_unique(ballot_words(positives, negatives), ballot_prefixes_negative)
}

/// Weak and bad filters partition a list of ballot words by length.
theorem weak_bad_filter_partition_length(words: List[List[Vote]]) {
    words.filter(ballot_prefixes_nonnegative).length + words.filter(ballot_prefixes_negative).length =
        words.length
} by {
    define p(items: List[List[Vote]]) -> Bool {
        items.filter(ballot_prefixes_nonnegative).length + items.filter(ballot_prefixes_negative).length =
            items.length
    }

    p(List.nil[List[Vote]])
    forall(head: List[Vote], tail: List[List[Vote]]) {
        if p(tail) {
            if ballot_prefixes_nonnegative(head) {
                List.cons(head, tail).filter(ballot_prefixes_nonnegative) =
                    List.cons(head, tail.filter(ballot_prefixes_nonnegative))
                List.cons(head, tail).filter(ballot_prefixes_nonnegative).length =
                    tail.filter(ballot_prefixes_nonnegative).length.suc
                List.cons(head, tail).filter(ballot_prefixes_negative).length =
                    tail.filter(ballot_prefixes_negative).length
                List.cons(head, tail).filter(ballot_prefixes_nonnegative).length +
                    List.cons(head, tail).filter(ballot_prefixes_negative).length =
                    (
                        tail.filter(ballot_prefixes_nonnegative).length +
                            tail.filter(ballot_prefixes_negative).length
                    ).suc
                tail.filter(ballot_prefixes_nonnegative).length + tail.filter(ballot_prefixes_negative).length =
                    tail.length
                p(List.cons(head, tail))
            }
            if not ballot_prefixes_nonnegative(head) {
                List.cons(head, tail).filter(ballot_prefixes_negative) =
                    List.cons(head, tail.filter(ballot_prefixes_negative))
                List.cons(head, tail).filter(ballot_prefixes_nonnegative).length =
                    tail.filter(ballot_prefixes_nonnegative).length
                List.cons(head, tail).filter(ballot_prefixes_negative).length =
                    tail.filter(ballot_prefixes_negative).length.suc
                List.cons(head, tail).filter(ballot_prefixes_nonnegative).length +
                    List.cons(head, tail).filter(ballot_prefixes_negative).length =
                    (
                        tail.filter(ballot_prefixes_nonnegative).length +
                            tail.filter(ballot_prefixes_negative).length
                    ).suc
                tail.filter(ballot_prefixes_nonnegative).length + tail.filter(ballot_prefixes_negative).length =
                    tail.length
                p(List.cons(head, tail))
            }
            p(List.cons(head, tail))
        }
    }

    p(words)
}

/// Weak and bad ballot words partition all generated ballot words by length.
theorem weak_bad_ballot_words_partition_length(positives: Nat, negatives: Nat) {
    weak_ballot_words(positives, negatives).length + bad_ballot_words(positives, negatives).length =
        ballot_words(positives, negatives).length
} by {
    weak_bad_filter_partition_length(ballot_words(positives, negatives))
}

/// The prepend-positive image of weak ballot words has no duplicates.
theorem strict_successful_tails_image_unique(positives: Nat, negatives: Nat) {
    strict_successful_tails_image(positives, negatives).is_unique
} by {
    weak_ballot_words_unique(positives, negatives)
    prepend_positive_injective
    injective_map_is_unique[List[Vote], List[Vote]](
        weak_ballot_words(positives, negatives),
        prepend_positive
    )
}

/// Strict successful ballots with one initial positive vote are counted by weak tails.
theorem successful_ballot_words_suc_length_eq_weak(positives: Nat, negatives: Nat) {
    successful_ballot_words(positives.suc, negatives).length =
        weak_ballot_words(positives, negatives).length
} by {
    strict_successful_tails_image_unique(positives, negatives)
    successful_ballot_words_unique(positives.suc, negatives)
    forall(word: List[Vote]) {
        strict_successful_tails_image_same_contains(positives, negatives, word)
        strict_successful_tails_image(positives, negatives).contains(word) =
            successful_ballot_words(positives.suc, negatives).contains(word)
    }
    unique_same_contains_imp_permutation[List[Vote]](
        strict_successful_tails_image(positives, negatives),
        successful_ballot_words(positives.suc, negatives)
    )
    permutation_preserves_length[List[Vote]](
        strict_successful_tails_image(positives, negatives),
        successful_ballot_words(positives.suc, negatives)
    )
    strict_successful_tails_image(positives, negatives).length =
        successful_ballot_words(positives.suc, negatives).length
    map_length[List[Vote], List[Vote]](weak_ballot_words(positives, negatives), prepend_positive)
}

/// The all-positive word has length equal to its number of positive votes.
theorem repeat_positive_length(n: Nat) {
    repeat_positive(n).length = n
} by {
    define p(k: Nat) -> Bool {
        repeat_positive(k).length = k
    }
    repeat_positive(Nat.0) = List.nil[Vote]
    List.nil[Vote].length = Nat.0
    repeat_positive(Nat.0).length = Nat.0
    p(Nat.0)
    forall(k: Nat) {
        if p(k) {
            repeat_positive(k.suc).length = repeat_positive(k).length.suc
            repeat_positive(k).length.suc = k.suc
            p(k.suc)
        }
    }
    p(n)
}

/// The all-positive word has exactly the prescribed number of positive votes.
theorem repeat_positive_positive_count(n: Nat) {
    positive_count(repeat_positive(n)) = n
} by {
    define p(k: Nat) -> Bool {
        positive_count(repeat_positive(k)) = k
    }
    repeat_positive(Nat.0) = List.nil[Vote]
    positive_count(List.nil[Vote]) = Nat.0
    positive_count(repeat_positive(Nat.0)) = Nat.0
    p(Nat.0) = (positive_count(repeat_positive(Nat.0)) = Nat.0)
    p(Nat.0)
    forall(k: Nat) {
        if p(k) {
            positive_count_cons_positive(repeat_positive(k))
            positive_count(repeat_positive(k)) = k
            p(k.suc)
        }
    }
    p(n)
}

/// The all-positive word has no negative votes.
theorem repeat_positive_negative_count(n: Nat) {
    negative_count(repeat_positive(n)) = Nat.0
} by {
    define p(k: Nat) -> Bool {
        negative_count(repeat_positive(k)) = Nat.0
    }
    repeat_positive(Nat.0) = List.nil[Vote]
    negative_count(List.nil[Vote]) = Nat.0
    negative_count(repeat_positive(Nat.0)) = Nat.0
    p(Nat.0)
    forall(k: Nat) {
        if p(k) {
            negative_count_cons_positive(repeat_positive(k))
            negative_count(repeat_positive(k.suc)) = negative_count(repeat_positive(k))
            p(k.suc)
        }
    }
    p(n)
}

/// The all-negative word has length equal to its number of negative votes.
theorem repeat_negative_length(n: Nat) {
    repeat_negative(n).length = n
} by {
    define p(k: Nat) -> Bool {
        repeat_negative(k).length = k
    }
    repeat_negative(Nat.0) = List.nil[Vote]
    List.nil[Vote].length = Nat.0
    repeat_negative(Nat.0).length = Nat.0
    p(Nat.0)
    forall(k: Nat) {
        if p(k) {
            repeat_negative(k.suc).length = repeat_negative(k).length.suc
            repeat_negative(k).length.suc = k.suc
            p(k.suc)
        }
    }
    p(n)
}

/// The all-negative word has no positive votes.
theorem repeat_negative_positive_count(n: Nat) {
    positive_count(repeat_negative(n)) = Nat.0
} by {
    define p(k: Nat) -> Bool {
        positive_count(repeat_negative(k)) = Nat.0
    }
    repeat_negative(Nat.0) = List.nil[Vote]
    positive_count(List.nil[Vote]) = Nat.0
    positive_count(repeat_negative(Nat.0)) = Nat.0
    p(Nat.0)
    forall(k: Nat) {
        if p(k) {
            positive_count_cons_negative(repeat_negative(k))
            positive_count(repeat_negative(k.suc)) = positive_count(repeat_negative(k))
            p(k.suc)
        }
    }
    p(n)
}

/// The all-negative word has exactly the prescribed number of negative votes.
theorem repeat_negative_negative_count(n: Nat) {
    negative_count(repeat_negative(n)) = n
} by {
    define p(k: Nat) -> Bool {
        negative_count(repeat_negative(k)) = k
    }
    repeat_negative(Nat.0) = List.nil[Vote]
    negative_count(List.nil[Vote]) = Nat.0
    negative_count(repeat_negative(Nat.0)) = Nat.0
    p(Nat.0)
    forall(k: Nat) {
        if p(k) {
            negative_count_cons_negative(repeat_negative(k))
            negative_count(repeat_negative(k)) = k
            p(k.suc)
        }
    }
    p(n)
}
