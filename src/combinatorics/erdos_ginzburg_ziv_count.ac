/// Counting lemmas for the Erdős–Ginzburg–Ziv theorem, case `n = 3`.
///
/// The residue classes modulo three of the five numbers are counted: either
/// some class occurs at least three times, or all three classes occur at
/// least once (three numbers from one class, or one from each class, then
/// sum to zero modulo three).

from nat import Nat, mod_lt, not_lt_zero, lt_suc_right, lt_cancel_suc, lt_imp_lte_suc,
    lte_cancel_suc, lt_or_lte, lte_imp_not_lt, lt_suc, lte_antisymm, sum_lte,
    add_cancels_left, only_zero_lte_zero, lte_trans, lte_add_left, lte_add_right
from list import List, map, sum, partial, map_length, length_range, add_length,
    list_not_contains_impl_count_zero, list_contains_implies_count_geq_one,
    map_contains, lt_of_range_contains, range_contains_of_lt

numerals Nat

/// The list of the residues of the first `m` values of `f` modulo `n`.
define residue_list(n: Nat, f: Nat -> Nat, m: Nat) -> List[Nat] {
    map(m.range, function(i: Nat) { f(i).mod(n) })
}

/// The residue list of the first `m` values has length `m`.
theorem residue_list_length(n: Nat, f: Nat -> Nat, m: Nat) {
    residue_list(n, f, m).length = m
} by {
    map_length(m.range, function(i: Nat) { f(i).mod(n) })
    map(m.range, function(i: Nat) { f(i).mod(n) }).length = m.range.length
    length_range(m)
    m.range.length = m
    residue_list(n, f, m).length = m
}

/// Every member of the residue list is below the modulus.
theorem residue_list_lt(n: Nat, f: Nat -> Nat, m: Nat, x: Nat) {
    n != Nat.0 and residue_list(n, f, m).contains(x) implies x < n
} by {
    if n != Nat.0 and residue_list(n, f, m).contains(x) {
        map_contains(m.range, function(i: Nat) { f(i).mod(n) }, x)
        let (i: Nat) satisfy {
            m.range.contains(i) and f(i).mod(n) = x
        }
        lt_of_range_contains(m, i)
        i < m
        mod_lt(f(i), n)
        n != Nat.0
        f(i).mod(n) < n
        f(i).mod(n) = x
        x < n
    }
}

/// A number below three is zero, one or two.
theorem lt_three_cases(x: Nat) {
    x < Nat.3 implies x = Nat.0 or x = Nat.1 or x = Nat.2
} by {
    if x < Nat.3 {
        lt_suc_right(x, Nat.2)
        x = Nat.2 or x < Nat.2
        if x = Nat.2 {
            x = Nat.0 or x = Nat.1 or x = Nat.2
        }
        if x < Nat.2 {
            if x < Nat.2 {
                lt_suc_right(x, Nat.1)
                x = Nat.1 or x < Nat.1
                if x = Nat.1 {
                    x = Nat.0 or x = Nat.1
                }
                if x < Nat.1 {
                    lt_suc_right(x, Nat.0)
                    x = Nat.0 or x < Nat.0
                    if x < Nat.0 {
                        not_lt_zero(x)
                        false
                    }
                    x = Nat.0 or x = Nat.1
                }
                x = Nat.0 or x = Nat.1
            }
            x = Nat.0 or x = Nat.1
            x = Nat.0 or x = Nat.1 or x = Nat.2
        }
        x = Nat.0 or x = Nat.1 or x = Nat.2
    }
}

/// Adding one to the left of both sides of one-at-most-two.
theorem one_lte_two_add(x: Nat) {
    Nat.1 + x <= Nat.2 + x
} by {
    lte_add_right(x, Nat.1, Nat.2)
    Nat.1 <= Nat.2
    Nat.1 + x <= Nat.2 + x
}
// /// The length of a list whose members are all `t` or `u` is at most
// /// the sum of the counts of `t` and `u`.
// theorem count_sum_bound(rs: List[Nat], t: Nat, u: Nat) {
//     forall(x: Nat) { rs.contains(x) implies x = t or x = u } implies
//     rs.length <= rs.count(t) + rs.count(u)
// } by {
//     define p(l: List[Nat]) -> Bool {
//         forall(x: Nat) { l.contains(x) implies x = t or x = u } implies
//         l.length <= l.count(t) + l.count(u)
//     }
//     if forall(x: Nat) { List.nil[Nat].contains(x) implies x = t or x = u } {
//         List.nil[Nat].length = Nat.0
//         List.nil[Nat].count(t) = Nat.0
//         List.nil[Nat].count(u) = Nat.0
//         Nat.0 <= Nat.0 + Nat.0
//         List.nil[Nat].length <= List.nil[Nat].count(t) + List.nil[Nat].count(u)
//     }
//     p(List.nil[Nat])
//     forall(head: Nat, tail: List[Nat]) {
//         if p(tail) {
//             if forall(x: Nat) { List.cons(head, tail).contains(x) implies x = t or x = u } {
//                 forall(x: Nat) {
//                     if tail.contains(x) {
//                         List.cons(head, tail).contains(x)
//                         x = t or x = u
//                     }
//                 }
//                 List.cons(head, tail).contains(head)
//                 head = t or head = u
//                 if head = t {
//                     if head = u {
//                         List.cons(head, tail).count(t) = Nat.1 + tail.count(t)
//                         List.cons(head, tail).count(u) = Nat.1 + tail.count(u)
//                         List.cons(head, tail).length = tail.length.suc
//                         List.cons(head, tail).length = Nat.1 + tail.length
//                         tail.length <= tail.count(t) + tail.count(u)
//                         lte_add_left(Nat.1, tail.length, tail.count(t) + tail.count(u))
//                         Nat.1 + tail.length <= Nat.1 + (tail.count(t) + tail.count(u))
//                         one_lte_two_add(tail.count(t) + tail.count(u))
//                         Nat.1 + (tail.count(t) + tail.count(u)) <= Nat.2 + (tail.count(t) + tail.count(u))
//                         lte_trans(Nat.1 + tail.length, Nat.1 + (tail.count(t) + tail.count(u)), Nat.2 + (tail.count(t) + tail.count(u)))
//                         Nat.1 + tail.length <= Nat.2 + (tail.count(t) + tail.count(u))
//                         Nat.2 + (tail.count(t) + tail.count(u)) = Nat.1 + tail.count(t) + Nat.1 + tail.count(u)
//                         Nat.1 + tail.count(t) + Nat.1 + tail.count(u) = List.cons(head, tail).count(t) + List.cons(head, tail).count(u)
//                         Nat.2 + (tail.count(t) + tail.count(u)) = List.cons(head, tail).count(t) + List.cons(head, tail).count(u)
//                         List.cons(head, tail).length <= List.cons(head, tail).count(t) + List.cons(head, tail).count(u)
//                     }
//                     if head != u {
//                         List.cons(head, tail).count(t) = Nat.1 + tail.count(t)
//                         List.cons(head, tail).count(u) = tail.count(u)
//                         List.cons(head, tail).length = tail.length.suc
//                         List.cons(head, tail).length = Nat.1 + tail.length
//                         tail.length <= tail.count(t) + tail.count(u)
//                         lte_add_left(Nat.1, tail.length, tail.count(t) + tail.count(u))
//                         Nat.1 + tail.length <= Nat.1 + (tail.count(t) + tail.count(u))
//                         List.cons(head, tail).count(t) + List.cons(head, tail).count(u) =
//                             Nat.1 + (tail.count(t) + tail.count(u))
//                         List.cons(head, tail).length <= List.cons(head, tail).count(t) + List.cons(head, tail).count(u)
//                     }
//                     List.cons(head, tail).length <= List.cons(head, tail).count(t) + List.cons(head, tail).count(u)
//                 }
//                 if head != t {
//                     if head = u {
//                         List.cons(head, tail).count(t) = tail.count(t)
//                         List.cons(head, tail).count(u) = Nat.1 + tail.count(u)
//                         List.cons(head, tail).length = tail.length.suc
//                         List.cons(head, tail).length = Nat.1 + tail.length
//                         tail.length <= tail.count(t) + tail.count(u)
//                         lte_add_left(Nat.1, tail.length, tail.count(t) + tail.count(u))
//                         Nat.1 + tail.length <= Nat.1 + (tail.count(t) + tail.count(u))
//                         List.cons(head, tail).count(t) + List.cons(head, tail).count(u) =
//                             Nat.1 + (tail.count(t) + tail.count(u))
//                         List.cons(head, tail).length <= List.cons(head, tail).count(t) + List.cons(head, tail).count(u)
//                     }
//                     if head != u {
//                         head = t or head = u
//                         false
//                     }
//                     List.cons(head, tail).length <= List.cons(head, tail).count(t) + List.cons(head, tail).count(u)
//                 }
//                 List.cons(head, tail).length <= List.cons(head, tail).count(t) + List.cons(head, tail).count(u)
//             }
//             p(List.cons(head, tail))
//         }
//     }
//     List.induction(function(rs2: List[Nat]) { p(rs2) })
//     forall(rs2: List[Nat]) { p(rs2) }
//     if forall(x: Nat) { rs.contains(x) implies x = t or x = u } {
//         rs.length <= rs.count(t) + rs.count(u)
//     }
//     p(rs)
// }

// /// Five elements in two classes: some class occurs at least three times.
// theorem five_into_two_classes(rs: List[Nat], t: Nat, u: Nat) {
//     rs.length = Nat.5 and forall(x: Nat) { rs.contains(x) implies x = t or x = u } implies
//     exists(v: Nat) {
//         rs.count(v) >= Nat.3
//     }
// } by {
//     if rs.length = Nat.5 and forall(x: Nat) { rs.contains(x) implies x = t or x = u } {
//         if rs.count(t) >= Nat.3 {
//             exists(v: Nat) {
//                 rs.count(v) >= Nat.3
//             }
//         }
//         if not rs.count(t) >= Nat.3 {
//             lt_or_lte(rs.count(t), Nat.3)
//             if Nat.3 <= rs.count(t) {
//                 false
//             }
//             rs.count(t) < Nat.3
//             lt_imp_lte_suc(rs.count(t), Nat.2)
//             rs.count(t) <= Nat.2
//             count_sum_bound(rs, t, u)
//             rs.length <= rs.count(t) + rs.count(u)
//             rs.length = Nat.5
//             Nat.5 <= rs.count(t) + rs.count(u)
//             sum_lte(rs.count(t), Nat.2, rs.count(u), rs.count(u))
//             rs.count(t) + rs.count(u) <= Nat.2 + rs.count(u)
//             lte_trans(Nat.5, rs.count(t) + rs.count(u), Nat.2 + rs.count(u))
//             Nat.5 <= Nat.2 + rs.count(u)
//             add_cancels_left(Nat.2, Nat.3, rs.count(u))
//             Nat.3 <= rs.count(u)
//             rs.count(u) >= Nat.3
//             exists(v: Nat) {
//                 rs.count(v) >= Nat.3
//             }
//         }
//         exists(v: Nat) {
//             rs.count(v) >= Nat.3
//         }
//     }
// }

// /// A five-element list of values below three either has some value at least
// /// three times, or contains all of zero, one and two.
// theorem three_classes_or_all_appear(rs: List[Nat]) {
//     rs.length = Nat.5 and forall(x: Nat) { rs.contains(x) implies x < Nat.3 } implies
//     (exists(v: Nat) {
//         rs.count(v) >= Nat.3
//     }) or
//     (rs.count(Nat.0) >= Nat.1 and rs.count(Nat.1) >= Nat.1 and rs.count(Nat.2) >= Nat.1)
// } by {
//     if rs.length = Nat.5 and forall(x: Nat) { rs.contains(x) implies x < Nat.3 } {
//         if exists(v: Nat) {
//             rs.count(v) >= Nat.3
//         } {
//             (exists(v: Nat) {
//                 rs.count(v) >= Nat.3
//             }) or
//             (rs.count(Nat.0) >= Nat.1 and rs.count(Nat.1) >= Nat.1 and rs.count(Nat.2) >= Nat.1)
//         }
//         if not exists(v: Nat) {
//             rs.count(v) >= Nat.3
//         } {
//             forall(v: Nat) {
//                 if rs.count(v) >= Nat.3 {
//                     exists(v2: Nat) {
//                         rs.count(v2) >= Nat.3
//                     }
//                     false
//                 }
//                 not rs.count(v) >= Nat.3
//             }
//             if rs.count(Nat.0) >= Nat.1 and rs.count(Nat.1) >= Nat.1 and rs.count(Nat.2) >= Nat.1 {
//                 (exists(v: Nat) {
//                     rs.count(v) >= Nat.3
//                 }) or
//                 (rs.count(Nat.0) >= Nat.1 and rs.count(Nat.1) >= Nat.1 and rs.count(Nat.2) >= Nat.1)
//             }
//             if not (rs.count(Nat.0) >= Nat.1 and rs.count(Nat.1) >= Nat.1 and rs.count(Nat.2) >= Nat.1) {
//                 if not rs.count(Nat.0) >= Nat.1 {
//                     lt_or_lte(rs.count(Nat.0), Nat.1)
//                     if Nat.1 <= rs.count(Nat.0) {
//                         false
//                     }
//                     rs.count(Nat.0) < Nat.1
//                     lt_imp_lte_suc(rs.count(Nat.0), Nat.0)
//                     rs.count(Nat.0) <= Nat.0
//                     only_zero_lte_zero(rs.count(Nat.0))
//                     rs.count(Nat.0) = Nat.0
//                     forall(x: Nat) {
//                         if rs.contains(x) {
//                             rs.contains(x) implies x < Nat.3
//                             x < Nat.3
//                             if x = Nat.0 {
//                                 rs.count(Nat.0) = Nat.0
//                                 list_contains_implies_count_geq_one(rs, x)
//                                 rs.contains(x) implies rs.count(x) >= Nat.1
//                                 rs.count(x) >= Nat.1
//                                 false
//                             }
//                             x != Nat.0
//                             x = Nat.1 or x = Nat.2
//                             x = Nat.1 or x = Nat.2
//                         }
//                         rs.contains(x) implies x = Nat.1 or x = Nat.2
//                     }
//                     five_into_two_classes(rs, Nat.1, Nat.2)
//                     rs.length = Nat.5 and forall(x: Nat) { rs.contains(x) implies x = Nat.1 or x = Nat.2 }
//                     exists(v: Nat) {
//                         rs.count(v) >= Nat.3
//                     }
//                     false
//                 }
//                 if rs.count(Nat.0) >= Nat.1 {
//                     if not rs.count(Nat.1) >= Nat.1 {
//                         false
//                     }
//                     if not rs.count(Nat.2) >= Nat.1 {
//                         false
//                     }
//                     rs.count(Nat.1) >= Nat.1 and rs.count(Nat.2) >= Nat.1
//                 }
//                 rs.count(Nat.0) >= Nat.1 and rs.count(Nat.1) >= Nat.1 and rs.count(Nat.2) >= Nat.1
//                 false
//             }
//             (exists(v: Nat) {
//                 rs.count(v) >= Nat.3
//             }) or
//             (rs.count(Nat.0) >= Nat.1 and rs.count(Nat.1) >= Nat.1 and rs.count(Nat.2) >= Nat.1)
//         }
//         (exists(v: Nat) {
//             rs.count(v) >= Nat.3
//         }) or
//         (rs.count(Nat.0) >= Nat.1 and rs.count(Nat.1) >= Nat.1 and rs.count(Nat.2) >= Nat.1)
//     }
// }

// /// The residue classes of the five values of `f` either contain three of a
// /// kind or contain all three classes.
// theorem egz_n3_counts(f: Nat -> Nat) {
//     (exists(v: Nat) {
//         residue_list(Nat.3, f, Nat.5).count(v) >= Nat.3
//     }) or
//     (residue_list(Nat.3, f, Nat.5).count(Nat.0) >= Nat.1 and
//      residue_list(Nat.3, f, Nat.5).count(Nat.1) >= Nat.1 and
//      residue_list(Nat.3, f, Nat.5).count(Nat.2) >= Nat.1)
// } by {
//     residue_list_length(Nat.3, f, Nat.5)
//     residue_list(Nat.3, f, Nat.5).length = Nat.5
//     forall(x: Nat) {
//         if residue_list(Nat.3, f, Nat.5).contains(x) {
//             residue_list_lt(Nat.3, f, Nat.5, x)
//             Nat.3 != Nat.0
//             x < Nat.3
//         }
//         residue_list(Nat.3, f, Nat.5).contains(x) implies x < Nat.3
//     }
//     three_classes_or_all_appear(residue_list(Nat.3, f, Nat.5))
//     residue_list(Nat.3, f, Nat.5).length = Nat.5 and
//         forall(x: Nat) { residue_list(Nat.3, f, Nat.5).contains(x) implies x < Nat.3 }
//     (exists(v: Nat) {
//         residue_list(Nat.3, f, Nat.5).count(v) >= Nat.3
//     }) or
//     (residue_list(Nat.3, f, Nat.5).count(Nat.0) >= Nat.1 and
//      residue_list(Nat.3, f, Nat.5).count(Nat.1) >= Nat.1 and
//      residue_list(Nat.3, f, Nat.5).count(Nat.2) >= Nat.1)
// }
