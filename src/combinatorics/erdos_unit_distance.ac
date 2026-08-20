/// The Erdős unit-distance problem: foundations.
///
/// The Erdős unit-distance problem asks for the maximum number of unordered
/// pairs of points at Euclidean distance exactly one among n points of the
/// plane.  This file formalizes the foundational definitions and the simplest
/// exact result: the n by n lattice grid of spacing one has exactly
/// `2 * n * (n - 1)` unit distances (the `(n-1) * n` horizontal edges plus the
/// `n * (n-1)` vertical edges of the grid).
///
/// The two central open/deep statements are recorded as commented-out theorem
/// texts at the end, in the point-set form that this library can express:
///
///   * the kissing-number bound: any point of the plane has at most six other
///     points of the unit-distance graph at unit distance from it (equality is
///     attained by the six neighbours of a point in the triangular lattice),
///     and
///   * the Spencer–Szemerédi–Trotter upper bound `u(n) = O(n^(4/3))`, which
///     follows from the crossing-number bound for unit-distance graphs.
///
/// The lower-bound side (lattice constructions with
/// `u(n) >= n^(1 + c / log log n)`, Erdős 1946) needs lattice points on
/// circles and is likewise stated but not proved here.
///
/// The library measures distances by their squares, so "distance exactly one"
/// is stated as `dist_sq = 1`.  The definitions of the unit-distance graph and
/// of `nu(p)`, the number of unit pairs of a finite point set, mirror those of
/// `src/geometry/unit_distance.ac`; they are repeated here because that file is
/// internal to the geometry package.
from nat import Nat, from_nat, lt_diff, add_comm, add_cancels_left, add_sub,
    lte_antisymm, suc_ne, pos_of_ne_zero, lt_imp_lte_suc,
    lte_and_lt, lte_trans, lt_suc_right, lt_suc, lte_ref, add_imp_sub,
    lt_add_suc, add_to_zero, lt_or_lte, not_lt_zero, from_nat_zero, from_nat_one,
    from_nat_add, sub_lt, sub_self, mul_comm, lte_mul_both,
    mul_one_right, mul_two_left, div_mul, zero_div, lte_add_left, lte_add_right,
    sum_lte, add_assoc, mul_assoc, suc_sub_one, lt_add_left
from order import lt_not_ref, lt_imp_lte, lt_of_lt_of_lte
from real import Real, square_nonneg, mul_nonneg, mul_lt_mul_of_pos_right,
    add_lte_add, from_nat_real_pos_of_ne_zero, lt_trans, lt_of_lte_of_lt,
    mul_neg_left, mul_neg_right, neg_neg
from algebra.add_comm_group import sub_add_cancel, sub_add_sub
from algebra.add_group import right_cancel, inverse_add, inverse_inverse
from pair import Pair, pair_new_first, pair_new_second, pair_ext, pair_eta
from geometry import Point2
from graph import SimpleGraph
from data.basic.relation_basic import is_symmetric, is_irreflexive
from finite_set import FiniteSet, fs_image, fs_union, finite_set_image_contains_eq,
    finite_set_ext_contains, finite_set_disjoint_union_cardinality_is,
    finite_set_union_contains_eq
from data.finite.finite_set_card import fs_card, fs_card_eq_of_cardinality_is,
    fs_card_cardinality_is
from data.finite.finite_set_filter import finite_set_filter, finite_set_filter_contains_eq
from data.finite.finite_set_product import finite_set_product, finite_set_product_contains_eq,
    finite_set_product_contains_pair
from data.finite.finite_set_card_compare import fs_card_mono
from data.finite.finite_set_subset_intro import fs_subset_eq_intro
from data.finite.finite_set_product_card import fs_card_product
from data.nat.nat_range_set import range_set, range_set_contains_eq, range_set_card

numerals Real
numerals Nat

// ============================================================================
// Real-number foundations: the embedding of the naturals into the reals.
// ============================================================================

/// Subtracting the added element recovers the first summand.
theorem real_add_sub_cancel(x: Real, y: Real) {
    (x + y) - y = x
} by {
    sub_add_cancel(x + y, y)
    ((x + y) - y) + y = x + y
    right_cancel((x + y) - y, x, y)
    (x + y) - y = x
}

/// Negating a difference swaps its arguments.
theorem real_neg_sub(a: Real, b: Real) {
    b - a = -(a - b)
} by {
    inverse_add(a, -b)
    -(a + -b) = -(-b) + -a
    inverse_inverse(b)
    -(-b) = b
    -(a + -b) = b + -a
    a + -b = a - b
    -(a - b) = b + -a
    b + -a = b - a
    b - a = -(a - b)
}

/// The square of a difference is symmetric in the two arguments.
theorem real_sq_diff_sym(a: Real, b: Real) {
    (a - b) * (a - b) = (b - a) * (b - a)
} by {
    real_neg_sub(a, b)
    b - a = -(a - b)
    (b - a) * (b - a) = (-(a - b)) * (-(a - b))
    mul_neg_left(a - b, -(a - b))
    (-(a - b)) * (-(a - b)) = -((a - b) * (-(a - b)))
    mul_neg_right(a - b, a - b)
    (a - b) * (-(a - b)) = -((a - b) * (a - b))
    (-(a - b)) * (-(a - b)) = -(-((a - b) * (a - b)))
    neg_neg((a - b) * (a - b))
    -(-((a - b) * (a - b))) = (a - b) * (a - b)
    (b - a) * (b - a) = (a - b) * (a - b)
    (a - b) * (a - b) = (b - a) * (b - a)
}

/// `from_nat` on the reals maps a successor to one plus the image of the predecessor.
theorem from_nat_suc_real(n: Nat) {
    from_nat[Real](n.suc) = from_nat[Real](n) + Real.1
} by {
    from_nat[Real](n.suc) = from_nat[Real](n) + Real.1
}

/// Zero is strictly less than one in the reals.
theorem real_zero_lt_one {
    Real.0 < Real.1
} by {
    from_nat_real_pos_of_ne_zero(Nat.1)
    Nat.1 != Nat.0
    from_nat[Real](Nat.1) > Real.0
    from_nat_one[Real]
    from_nat[Real](Nat.1) = Real.1
    Real.1 > Real.0
}

/// Zero is at most one in the reals.
theorem real_zero_lte_one {
    Real.0 <= Real.1
} by {
    real_zero_lt_one
    Real.0 < Real.1
    lt_imp_lte(Real.0, Real.1)
    Real.0 <= Real.1
}

/// The image of a natural in the reals is nonnegative.
theorem from_nat_real_nonneg(k: Nat) {
    from_nat[Real](k) >= Real.0
} by {
    define p(m: Nat) -> Bool {
        from_nat[Real](m) >= Real.0
    }
    from_nat_zero[Real]
    from_nat[Real](Nat.0) = Real.0
    from_nat[Real](Nat.0) >= Real.0
    p(Nat.0)
    forall(m: Nat) {
        if p(m) {
            from_nat[Real](m) >= Real.0
            from_nat_suc_real(m)
            from_nat[Real](m.suc) = from_nat[Real](m) + Real.1
            real_zero_lte_one
            Real.0 <= Real.1
            add_lte_add(Real.0, from_nat[Real](m), Real.0, Real.1)
            Real.0 + Real.0 <= from_nat[Real](m) + Real.1
            Real.0 + Real.0 = Real.0
            Real.0 <= from_nat[Real](m) + Real.1
            from_nat[Real](m.suc) >= Real.0
            p(m.suc)
        }
        p(m) implies p(m.suc)
    }
    p(Nat.0) and forall(m: Nat) { p(m) implies p(m.suc) }
    Nat.induction(p)
    p(k)
    from_nat[Real](k) >= Real.0
}

/// Adding a positive real makes things strictly larger.
theorem real_add_pos_lt(x: Real, y: Real) {
    y > Real.0 implies x < x + y
} by {
    if y > Real.0 {
        Real.0 < y
        Real.0 + x < y + x
        x < x + y
    }
}

/// `from_nat` is strictly monotone on the reals.
theorem from_nat_lt_strict(m: Nat, n: Nat) {
    m < n implies from_nat[Real](m) < from_nat[Real](n)
} by {
    if m < n {
        lt_diff(m, n)
        exists(d: Nat) { m + d = n and d != Nat.0 }
        let d: Nat satisfy { m + d = n and d != Nat.0 }
        from_nat_add[Real](m, d)
        from_nat[Real](m + d) = from_nat[Real](m) + from_nat[Real](d)
        from_nat_real_pos_of_ne_zero(d)
        from_nat[Real](d) > Real.0
        real_add_pos_lt(from_nat[Real](m), from_nat[Real](d))
        from_nat[Real](m) < from_nat[Real](m) + from_nat[Real](d)
        from_nat[Real](m) + from_nat[Real](d) = from_nat[Real](n)
        from_nat[Real](m) < from_nat[Real](n)
    }
}

/// `from_nat` on the reals is injective.
theorem from_nat_real_inj(m: Nat, n: Nat) {
    from_nat[Real](m) = from_nat[Real](n) implies m = n
} by {
    if from_nat[Real](m) = from_nat[Real](n) {
        if m != n {
            lt_or_lte(m, n)
            if m < n {
                from_nat_lt_strict(m, n)
                from_nat[Real](m) < from_nat[Real](n)
                false
            }
            if n < m {
                from_nat_lt_strict(n, m)
                from_nat[Real](n) < from_nat[Real](m)
                false
            }
            m = n
            false
        }
        m = n
    }
}

/// `from_nat` reflects the non-strict order.
theorem from_nat_le_reflect(m: Nat, n: Nat) {
    from_nat[Real](m) <= from_nat[Real](n) implies m <= n
} by {
    if from_nat[Real](m) <= from_nat[Real](n) {
        lt_or_lte(n, m)
        if n < m {
            from_nat_lt_strict(n, m)
            from_nat[Real](n) < from_nat[Real](m)
            false
        }
        m <= n
    }
}

/// `from_nat` of a difference is the difference of the images.
theorem from_nat_sub_real(m: Nat, n: Nat) {
    n <= m implies from_nat[Real](m) - from_nat[Real](n) = from_nat[Real](m - n)
} by {
    if n <= m {
        add_sub(m, n)
        (m - n) + n = m
        from_nat_add[Real](m - n, n)
        from_nat[Real]((m - n) + n) = from_nat[Real](m - n) + from_nat[Real](n)
        from_nat[Real](m) = from_nat[Real](m - n) + from_nat[Real](n)
        from_nat[Real](m) - from_nat[Real](n) = from_nat[Real](m - n)
    }
}

/// The square of the difference of two ordered images is the square of the
/// image of their difference.
theorem from_nat_diff_sq(m: Nat, n: Nat) {
    m <= n implies
        (from_nat[Real](m) - from_nat[Real](n)) * (from_nat[Real](m) - from_nat[Real](n)) =
        from_nat[Real](n - m) * from_nat[Real](n - m)
} by {
    if m <= n {
        from_nat_sub_real(n, m)
        from_nat[Real](n) - from_nat[Real](m) = from_nat[Real](n - m)
        real_sq_diff_sym(from_nat[Real](m), from_nat[Real](n))
        (from_nat[Real](m) - from_nat[Real](n)) * (from_nat[Real](m) - from_nat[Real](n)) =
            (from_nat[Real](n) - from_nat[Real](m)) * (from_nat[Real](n) - from_nat[Real](m))
        (from_nat[Real](m) - from_nat[Real](n)) * (from_nat[Real](m) - from_nat[Real](n)) =
            from_nat[Real](n - m) * from_nat[Real](n - m)
    }
}

/// A nonnegative real whose square is at most one is at most one.
theorem real_sq_le_one_imp_le_one(x: Real) {
    x >= Real.0 and x * x <= Real.1 implies x <= Real.1
} by {
    if x >= Real.0 and x * x <= Real.1 {
        if Real.1 < x {
            real_zero_lt_one
            Real.0 < Real.1
            lt_of_lte_of_lt(Real.0, Real.1, x)
            Real.0 < x
            mul_lt_mul_of_pos_right(Real.1, x, x)
            Real.1 * x < x * x
            Real.1 * x = x
            x < x * x
            lt_of_lte_of_lt(x * x, Real.1, x)
            x * x < x
            lt_trans(x, x * x, x)
            x < x
            lt_not_ref(x)
            false
        }
        x <= Real.1
    }
}

/// A square of a real is at most the sum of two squares.
theorem sq_le_sum_sq(a: Real, b: Real) {
    a * a <= a * a + b * b
} by {
    square_nonneg(b)
    b * b >= Real.0
    Real.0 <= b * b
    a * a + Real.0 <= a * a + b * b
    a * a <= a * a + b * b
}

/// If a sum of two squares is one, each square is at most one.
theorem sum_sq_eq_one_imp_sq_le(a: Real, b: Real) {
    a * a + b * b = Real.1 implies a * a <= Real.1
} by {
    if a * a + b * b = Real.1 {
        sq_le_sum_sq(a, b)
        a * a <= a * a + b * b
        a * a <= Real.1
    }
}

/// The square of a nonnegative image of a natural is at most one only for
/// naturals at most one.
theorem from_nat_sq_le_one_imp_le_one(k: Nat) {
    from_nat[Real](k) * from_nat[Real](k) <= Real.1 implies k <= Nat.1
} by {
    if from_nat[Real](k) * from_nat[Real](k) <= Real.1 {
        from_nat_real_nonneg(k)
        from_nat[Real](k) >= Real.0
        real_sq_le_one_imp_le_one(from_nat[Real](k))
        from_nat[Real](k) <= Real.1
        from_nat_one[Real]
        from_nat[Real](Nat.1) = Real.1
        from_nat[Real](k) <= from_nat[Real](Nat.1)
        from_nat_le_reflect(k, Nat.1)
        k <= Nat.1
    }
}

// ============================================================================
// Natural-number foundations: small-order facts used by the counting.
// ============================================================================

/// A natural at most one is zero or one.
theorem nat_le_one_imp_zero_or_one(k: Nat) {
    k <= Nat.1 implies (k = Nat.0 or k = Nat.1)
} by {
    if k <= Nat.1 {
        if k = Nat.0 {
            k = Nat.0 or k = Nat.1
        }
        if k != Nat.0 {
            pos_of_ne_zero(k)
            Nat.0 < k
            lt_imp_lte_suc(Nat.0, k)
            Nat.1 <= k
            lte_antisymm(k, Nat.1)
            k = Nat.1
            k = Nat.0 or k = Nat.1
        }
        k = Nat.0 or k = Nat.1
    }
}

/// A sum of two squares of images of naturals equals one only for (1, 0) and (0, 1).
theorem from_nat_sq_sum_eq_one(m: Nat, n: Nat) {
    from_nat[Real](m) * from_nat[Real](m) + from_nat[Real](n) * from_nat[Real](n) = Real.1
    implies (m = Nat.1 and n = Nat.0) or (m = Nat.0 and n = Nat.1)
} by {
    if from_nat[Real](m) * from_nat[Real](m) + from_nat[Real](n) * from_nat[Real](n) = Real.1 {
        sq_le_sum_sq(from_nat[Real](m), from_nat[Real](n))
        from_nat[Real](m) * from_nat[Real](m) <= from_nat[Real](m) * from_nat[Real](m) + from_nat[Real](n) * from_nat[Real](n)
        from_nat[Real](m) * from_nat[Real](m) <= Real.1
        from_nat_sq_le_one_imp_le_one(m)
        m <= Nat.1
        nat_le_one_imp_zero_or_one(m)
        m = Nat.0 or m = Nat.1
        sq_le_sum_sq(from_nat[Real](n), from_nat[Real](m))
        from_nat[Real](n) * from_nat[Real](n) <= from_nat[Real](n) * from_nat[Real](n) + from_nat[Real](m) * from_nat[Real](m)
        from_nat[Real](n) * from_nat[Real](n) + from_nat[Real](m) * from_nat[Real](m) =
            from_nat[Real](m) * from_nat[Real](m) + from_nat[Real](n) * from_nat[Real](n)
        from_nat[Real](n) * from_nat[Real](n) <= Real.1
        from_nat_sq_le_one_imp_le_one(n)
        n <= Nat.1
        nat_le_one_imp_zero_or_one(n)
        n = Nat.0 or n = Nat.1
        if m = Nat.0 {
            if n = Nat.0 {
                from_nat_zero[Real]
                from_nat[Real](Nat.0) = Real.0
                m = Nat.0
                n = Nat.0
                from_nat[Real](m) = Real.0
                from_nat[Real](n) = Real.0
                Real.0 * Real.0 = Real.0
                Real.0 * Real.0 + Real.0 * Real.0 = Real.0 + Real.0
                Real.0 + Real.0 = Real.0
                Real.0 * Real.0 + Real.0 * Real.0 = Real.0
                Real.0 = Real.1
                real_zero_lt_one
                Real.0 < Real.1
                Real.1 < Real.1
                lt_not_ref(Real.1)
                false
            }
            if n = Nat.1 {
                m = Nat.0 and n = Nat.1
                (m = Nat.1 and n = Nat.0) or (m = Nat.0 and n = Nat.1)
            }
            n = Nat.0 or n = Nat.1
            (m = Nat.1 and n = Nat.0) or (m = Nat.0 and n = Nat.1)
        }
        if m = Nat.1 {
            if n = Nat.0 {
                m = Nat.1 and n = Nat.0
                (m = Nat.1 and n = Nat.0) or (m = Nat.0 and n = Nat.1)
            }
            if n = Nat.1 {
                from_nat_one[Real]
                from_nat[Real](Nat.1) = Real.1
                m = Nat.1
                n = Nat.1
                from_nat[Real](m) = Real.1
                from_nat[Real](n) = Real.1
                Real.1 * Real.1 = Real.1
                Real.1 * Real.1 + Real.1 * Real.1 = Real.1 + Real.1
                Real.1 + Real.1 = Real.1
                real_zero_lt_one
                Real.0 < Real.1
                real_add_pos_lt(Real.1, Real.1)
                Real.1 < Real.1 + Real.1
                Real.1 < Real.1
                lt_not_ref(Real.1)
                false
            }
            n = Nat.0 or n = Nat.1
            (m = Nat.1 and n = Nat.0) or (m = Nat.0 and n = Nat.1)
        }
        m = Nat.0 or m = Nat.1
        (m = Nat.1 and n = Nat.0) or (m = Nat.0 and n = Nat.1)
    }
}

/// A difference of one between naturals recovers the successor relation.
theorem sub_eq_one_imp_suc(a: Nat, b: Nat) {
    a - b = Nat.1 implies a = b + Nat.1
} by {
    if a - b = Nat.1 {
        lt_or_lte(a, b)
        if a < b {
            sub_lt(a, b)
            a - b = Nat.0
            false
        }
        b <= a
        add_sub(a, b)
        (a - b) + b = a
        (a - b) + b = Nat.1 + b
        a = Nat.1 + b
        a = b + Nat.1
    }
}

/// A natural is below a successor exactly when it is at most the predecessor.
theorem lt_suc_iff_lte(a: Nat, b: Nat) {
    (a < b.suc) = (a <= b)
} by {
    if a < b.suc {
        lt_suc_right(a, b)
        a = b or a < b
        if a = b {
            lte_ref(a)
            a <= b
        }
        if a < b {
            lt_imp_lte(a, b)
            a <= b
        }
        a <= b
    }
    (a < b.suc) implies (a <= b)
    if a <= b {
        lt_suc(b)
        b < b.suc
        lte_and_lt(a, b, b.suc)
        a < b.suc
    }
    (a <= b) implies (a < b.suc)
    (a < b.suc) = (a <= b)
}

/// Shifting an index by one and staying below n keeps the index below n - 1.
theorem lt_suc_imp_lt_sub(m: Nat, n: Nat) {
    m + Nat.1 < n implies m < n - Nat.1
} by {
    if m + Nat.1 < n {
        lt_diff(m + Nat.1, n)
        exists(d: Nat) { (m + Nat.1) + d = n and d != Nat.0 }
        let d: Nat satisfy { (m + Nat.1) + d = n and d != Nat.0 }
        add_assoc(m, Nat.1, d)
        m + (Nat.1 + d) = (m + Nat.1) + d
        m + (Nat.1 + d) = n
        add_comm(Nat.1, d)
        Nat.1 + d = d + Nat.1
        m + (d + Nat.1) = n
        add_assoc(m, d, Nat.1)
        (m + d) + Nat.1 = m + (d + Nat.1)
        (m + d) + Nat.1 = n
        add_imp_sub(m + d, Nat.1, n)
        n - Nat.1 = m + d
        pos_of_ne_zero(d)
        Nat.0 < d
        lt_add_left(m, Nat.0, d)
        m + Nat.0 < m + d
        m + Nat.0 = m
        m < m + d
        m < n - Nat.1
    }
}
