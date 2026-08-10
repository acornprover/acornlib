/// The classical partition identities.
///
/// This file records the classical identities for the partition function
/// `p` of `combinatorics.partitions`:
///
///   (a) a partition of `n` has all parts at most `n`, so the partitions of
///       `n` into parts of size at most `n` are exactly the partitions of
///       `n`, and `p(n)` counts them;
///   (b) Euler's identity: the number of partitions of `n` into odd parts
///       equals the number into distinct parts, proved here for `n = 4`
///       (both numbers are 2);
///   (c) `p` grows: `p(n + 1) > p(n)` for `n >= 1`, proved here for the
///       small values 1, 2, 3;
///   (d) Euler's pentagonal number theorem: `p` satisfies the pentagonal
///       recurrence, stated here and verified for `n = 4`.

from nat import Nat, add_comm, add_imp_sub, sub_self, lt_suc, lt_trans
from list import List, sum, unique_implies_tail_unique, singleton_unique,
    cons_unique_of_tail_unique_not_contains
from data.basic.logic import or_intro_left, or_intro_right
from data.list.list_cons_membership import cons_contains_eq, cons_contains_of_tail_contains,
    cons_contains_head, nil_not_contains
from data.list.list_unique_cons import unique_cons_head_not_in_tail
from data.basic.set import Set, contains_set_intro, set_eq_of_contains_at_eq,
    set_eq_transport_predicate
from combinatorics.partitions import is_partition, is_partition_sum, partition_set,
    partition_set_contains_iff, p, partition_four_char, p4_a, p4_b, p4_c, p4_d,
    p4_e, p4_sound, pair_is_unique, cons_neq_head, bool_eq_of_iff, contains_le_sum,
    filter_all_keep, p_one, p_two, p_three, p_four, partition_set_finite

numerals Nat

/// True if the natural number `n` is odd.
define is_odd(n: Nat) -> Bool {
    match n {
        Nat.zero {
            false
        }
        Nat.suc(pred) {
            not is_odd(pred)
        }
    }
}

/// Oddness flips under successor.
theorem is_odd_suc(k: Nat) {
    is_odd(k.suc) = not is_odd(k)
}

/// Zero is even.
theorem is_odd_zero {
    is_odd(Nat.0) = false
}

/// One is odd.
theorem is_odd_one {
    is_odd(Nat.1) = true
}

/// Two is even.
theorem is_odd_two {
    is_odd(Nat.2) = false
}

/// Three is odd.
theorem is_odd_three {
    is_odd(Nat.3) = true
}

/// Four is even.
theorem is_odd_four {
    is_odd(Nat.4) = false
}

/// True if every element of the list is odd.
define all_odd(parts: List[Nat]) -> Bool {
    match parts {
        List.nil {
            true
        }
        List.cons(head, tail) {
            is_odd(head) and all_odd(tail)
        }
    }
}

/// A partition of `n` has all parts at most `n`.
theorem partition_parts_at_most_n(parts: List[Nat], n: Nat) {
    is_partition(parts, n) implies forall(x: Nat) { parts.contains(x) implies x <= n }
} by {
    if is_partition(parts, n) {
        is_partition_sum(parts, n)
        sum(parts) = n
        forall(x: Nat) {
            if parts.contains(x) {
                contains_le_sum(parts, x)
                parts.contains(x) implies x <= sum(parts)
                x <= sum(parts)
                x <= n
            }
        }
        forall(x: Nat) { parts.contains(x) implies x <= n }
    }
}

/// True if `parts` is a partition of `n` into parts of size at most `n`.
define is_partition_at_most(parts: List[Nat], n: Nat) -> Bool {
    is_partition(parts, n) and forall(x: Nat) { parts.contains(x) implies x <= n }
}

/// The partitions of `n` into parts of size at most `n` are exactly the
/// partitions of `n`.
theorem partition_at_most_char(parts: List[Nat], n: Nat) {
    is_partition_at_most(parts, n) = is_partition(parts, n)
} by {
    if is_partition_at_most(parts, n) {
        is_partition(parts, n)
    }
    is_partition_at_most(parts, n) implies is_partition(parts, n)
    if is_partition(parts, n) {
        partition_parts_at_most_n(parts, n)
        is_partition(parts, n) implies forall(x: Nat) { parts.contains(x) implies x <= n }
        forall(x: Nat) { parts.contains(x) implies x <= n }
        is_partition(parts, n) and forall(x: Nat) { parts.contains(x) implies x <= n }
        is_partition_at_most(parts, n)
    }
    is_partition(parts, n) implies is_partition_at_most(parts, n)
    (is_partition_at_most(parts, n) implies is_partition(parts, n)) and
        (is_partition(parts, n) implies is_partition_at_most(parts, n))
    bool_eq_of_iff(is_partition_at_most(parts, n), is_partition(parts, n))
    is_partition_at_most(parts, n) = is_partition(parts, n)
}

/// The membership predicate of the set of partitions of `n` into parts of
/// size at most `n`.
define is_partition_at_most_of_n(n: Nat, parts: List[Nat]) -> Bool {
    is_partition_at_most(parts, n)
}

/// The set of partitions of `n` into parts of size at most `n`.
define partition_at_most_set(n: Nat) -> Set[List[Nat]] {
    Set.new(is_partition_at_most_of_n(n))
}

/// Membership in the set of partitions of `n` into parts of size at most `n`
/// means being a partition of `n`.
theorem partition_at_most_set_contains_iff(n: Nat, parts: List[Nat]) {
    partition_at_most_set(n).contains(parts) = is_partition(parts, n)
} by {
    partition_at_most_set(n).contains(parts) = is_partition_at_most_of_n(n, parts)
    is_partition_at_most_of_n(n, parts) = is_partition_at_most(parts, n)
    partition_at_most_char(parts, n)
    is_partition_at_most(parts, n) = is_partition(parts, n)
    partition_at_most_set(n).contains(parts) = is_partition(parts, n)
}

/// The set of partitions of `n` into parts of size at most `n` is the set of
/// partitions of `n`.
theorem partition_at_most_set_eq(n: Nat) {
    partition_at_most_set(n) = partition_set(n)
} by {
    forall(parts: List[Nat]) {
        partition_at_most_set_contains_iff(n, parts)
        partition_at_most_set(n).contains(parts) = is_partition(parts, n)
        partition_set_contains_iff(n, parts)
        partition_set(n).contains(parts) = is_partition(parts, n)
        partition_at_most_set(n).contains(parts) = partition_set(n).contains(parts)
    }
    forall(parts: List[Nat]) {
        partition_at_most_set(n).contains(parts) = partition_set(n).contains(parts)
    }
    set_eq_of_contains_at_eq(partition_at_most_set(n), partition_set(n))
    partition_at_most_set(n) = partition_set(n)
}

/// The set of partitions of `n` into parts of size at most `n` is finite.
theorem partition_at_most_set_finite(n: Nat) {
    partition_at_most_set(n).is_finite
} by {
    partition_at_most_set_eq(n)
    partition_at_most_set(n) = partition_set(n)
    partition_set_finite(n)
    partition_set(n).is_finite
    set_eq_transport_predicate(function(s: Set[List[Nat]]) { s.is_finite }, partition_set(n), partition_at_most_set(n))
    partition_set(n) = partition_at_most_set(n) and partition_set(n).is_finite implies partition_at_most_set(n).is_finite
    partition_set(n) = partition_at_most_set(n)
    partition_set(n).is_finite
    partition_at_most_set(n).is_finite
}

/// `p(n)` is the number of partitions of `n` into parts of size at most `n`.
theorem p_counts_parts_at_most_n(n: Nat) {
    partition_at_most_set(n).cardinality_is(p(n))
} by {
    partition_at_most_set_eq(n)
    partition_at_most_set(n) = partition_set(n)
    partition_set(n).cardinality_is(p(n))
    set_eq_transport_predicate(function(s: Set[List[Nat]]) { s.cardinality_is(p(n)) }, partition_set(n), partition_at_most_set(n))
    partition_set(n) = partition_at_most_set(n) and partition_set(n).cardinality_is(p(n)) implies partition_at_most_set(n).cardinality_is(p(n))
    partition_set(n) = partition_at_most_set(n)
    partition_set(n).cardinality_is(p(n))
    partition_at_most_set(n).cardinality_is(p(n))
}

// ---------------------------------------------------------------------------
// Euler's identity: partitions into odd parts and into distinct parts.
// ---------------------------------------------------------------------------
//
// Euler's identity says that for every `n` the number of partitions of `n`
// into odd parts equals the number of partitions of `n` into distinct parts.
// The classical proof is a bijection (Glaisher's map, or the "odd versus
// distinct" doubling map); the general statement is recorded at the bottom of
// the file.  Here the identity is proved for `n = 4`, where both numbers are
// two: the odd-part partitions are 3 + 1 and 1 + 1 + 1 + 1, and the
// distinct-part partitions are 4 and 3 + 1.

/// True if `parts` is a partition of `n` into odd parts.
define is_odd_part_partition(parts: List[Nat], n: Nat) -> Bool {
    is_partition(parts, n) and all_odd(parts)
}

/// True if `parts` is a partition of `n` into distinct parts.
define is_distinct_part_partition(parts: List[Nat], n: Nat) -> Bool {
    is_partition(parts, n) and parts.is_unique
}

/// The membership predicate of the set of odd-part partitions of `n`.
define is_odd_part_partition_of_n(n: Nat, parts: List[Nat]) -> Bool {
    is_odd_part_partition(parts, n)
}

/// The membership predicate of the set of distinct-part partitions of `n`.
define is_distinct_part_partition_of_n(n: Nat, parts: List[Nat]) -> Bool {
    is_distinct_part_partition(parts, n)
}

/// The set of partitions of `n` into odd parts.
define odd_partition_set(n: Nat) -> Set[List[Nat]] {
    Set.new(is_odd_part_partition_of_n(n))
}

/// The set of partitions of `n` into distinct parts.
define distinct_partition_set(n: Nat) -> Set[List[Nat]] {
    Set.new(is_distinct_part_partition_of_n(n))
}

/// Membership in the set of odd-part partitions of `n`.
theorem odd_partition_set_contains_iff(n: Nat, parts: List[Nat]) {
    odd_partition_set(n).contains(parts) = is_odd_part_partition(parts, n)
}

/// Membership in the set of distinct-part partitions of `n`.
theorem distinct_partition_set_contains_iff(n: Nat, parts: List[Nat]) {
    distinct_partition_set(n).contains(parts) = is_distinct_part_partition(parts, n)
}

/// The list 3, 1 is all-odd.
theorem all_odd_31 {
    all_odd(List.cons(Nat.3, List.cons(Nat.1, List.nil[Nat]))) = true
} by {
    is_odd_one
    is_odd(Nat.1) = true
    is_odd_three
    is_odd(Nat.3) = true
    all_odd(List.cons(Nat.3, List.cons(Nat.1, List.nil[Nat]))) = true
}

/// The list 1, 1, 1, 1 is all-odd.
theorem all_odd_1111 {
    all_odd(List.cons(Nat.1, List.cons(Nat.1, List.cons(Nat.1, List.cons(Nat.1, List.nil[Nat]))))) = true
} by {
    is_odd_one
    is_odd(Nat.1) = true
    all_odd(List.cons(Nat.1, List.nil[Nat])) = true
    all_odd(List.cons(Nat.1, List.cons(Nat.1, List.nil[Nat]))) = true
    all_odd(List.cons(Nat.1, List.cons(Nat.1, List.cons(Nat.1, List.nil[Nat])))) = true
    all_odd(List.cons(Nat.1, List.cons(Nat.1, List.cons(Nat.1, List.cons(Nat.1, List.nil[Nat]))))) = true
}

/// The singleton 4 is not all-odd.
theorem all_odd_4 {
    all_odd(List.cons(Nat.4, List.nil[Nat])) = false
} by {
    is_odd_four
    is_odd(Nat.4) = false
    all_odd(List.cons(Nat.4, List.nil[Nat])) = false
}

/// The list 2, 2 is not all-odd.
theorem all_odd_22 {
    all_odd(List.cons(Nat.2, List.cons(Nat.2, List.nil[Nat]))) = false
} by {
    is_odd_two
    is_odd(Nat.2) = false
    all_odd(List.cons(Nat.2, List.cons(Nat.2, List.nil[Nat]))) = false
}

/// The list 2, 1, 1 is not all-odd.
theorem all_odd_211 {
    all_odd(List.cons(Nat.2, List.cons(Nat.1, List.cons(Nat.1, List.nil[Nat])))) = false
} by {
    is_odd_two
    is_odd(Nat.2) = false
    all_odd(List.cons(Nat.2, List.cons(Nat.1, List.cons(Nat.1, List.nil[Nat])))) = false
}

/// A two-element list of distinct naturals is unique.
theorem pair_nat_unique(a: Nat, b: Nat) {
    a != b implies List.cons(a, List.cons(b, List.nil[Nat])).is_unique
} by {
    if a != b {
        singleton_unique(b)
        List.singleton(b).is_unique
        List.cons(b, List.nil[Nat]) = List.singleton(b)
        List.cons(b, List.nil[Nat]).is_unique
        nil_not_contains(b)
        not List.nil[Nat].contains(b)
        cons_unique_of_tail_unique_not_contains(b, List.nil[Nat])
        List.nil[Nat].is_unique and not List.nil[Nat].contains(b) implies List.cons(b, List.nil[Nat]).is_unique
        List.nil[Nat].is_unique
        List.nil[Nat].is_unique and not List.nil[Nat].contains(b)
        List.cons(b, List.nil[Nat]).is_unique
        cons_contains_eq(b, List.nil[Nat], a)
        List.cons(b, List.nil[Nat]).contains(a) =
            (b = a or List.nil[Nat].contains(a))
        b != a
        not List.nil[Nat].contains(a)
        not List.cons(b, List.nil[Nat]).contains(a)
        cons_unique_of_tail_unique_not_contains(a, List.cons(b, List.nil[Nat]))
        List.cons(b, List.nil[Nat]).is_unique and not List.cons(b, List.nil[Nat]).contains(a) implies List.cons(a, List.cons(b, List.nil[Nat])).is_unique
        List.cons(b, List.nil[Nat]).is_unique and not List.cons(b, List.nil[Nat]).contains(a)
        List.cons(a, List.cons(b, List.nil[Nat])).is_unique
    }
}

/// A list containing an element more than once is not unique: if the head
/// reappears in the tail, the tail already contains it.
theorem not_unique_22 {
    not List.cons(Nat.2, List.cons(Nat.2, List.nil[Nat])).is_unique
} by {
    if List.cons(Nat.2, List.cons(Nat.2, List.nil[Nat])).is_unique {
        unique_cons_head_not_in_tail(Nat.2, List.cons(Nat.2, List.nil[Nat]))
        List.cons(Nat.2, List.cons(Nat.2, List.nil[Nat])).is_unique implies not List.cons(Nat.2, List.nil[Nat]).contains(Nat.2)
        not List.cons(Nat.2, List.nil[Nat]).contains(Nat.2)
        cons_contains_head(Nat.2, List.nil[Nat])
        List.cons(Nat.2, List.nil[Nat]).contains(Nat.2)
        false
    }
    not List.cons(Nat.2, List.cons(Nat.2, List.nil[Nat])).is_unique
}

/// The list 2, 1, 1 is not unique: its tail 1, 1 repeats its head 1.
theorem not_unique_211 {
    not List.cons(Nat.2, List.cons(Nat.1, List.cons(Nat.1, List.nil[Nat]))).is_unique
} by {
    if List.cons(Nat.2, List.cons(Nat.1, List.cons(Nat.1, List.nil[Nat]))).is_unique {
        unique_implies_tail_unique(Nat.2, List.cons(Nat.1, List.cons(Nat.1, List.nil[Nat])))
        List.cons(Nat.2, List.cons(Nat.1, List.cons(Nat.1, List.nil[Nat]))).is_unique implies List.cons(Nat.1, List.cons(Nat.1, List.nil[Nat])).is_unique
        List.cons(Nat.1, List.cons(Nat.1, List.nil[Nat])).is_unique
        unique_cons_head_not_in_tail(Nat.1, List.cons(Nat.1, List.nil[Nat]))
        List.cons(Nat.1, List.cons(Nat.1, List.nil[Nat])).is_unique implies not List.cons(Nat.1, List.nil[Nat]).contains(Nat.1)
        not List.cons(Nat.1, List.nil[Nat]).contains(Nat.1)
        cons_contains_head(Nat.1, List.nil[Nat])
        List.cons(Nat.1, List.nil[Nat]).contains(Nat.1)
        false
    }
    not List.cons(Nat.2, List.cons(Nat.1, List.cons(Nat.1, List.nil[Nat]))).is_unique
}

/// The list 1, 1, 1, 1 is not unique: its tail 1, 1, 1 repeats its head 1.
theorem not_unique_1111 {
    not List.cons(Nat.1, List.cons(Nat.1, List.cons(Nat.1, List.cons(Nat.1, List.nil[Nat])))).is_unique
} by {
    if List.cons(Nat.1, List.cons(Nat.1, List.cons(Nat.1, List.cons(Nat.1, List.nil[Nat])))).is_unique {
        unique_implies_tail_unique(Nat.1, List.cons(Nat.1, List.cons(Nat.1, List.cons(Nat.1, List.nil[Nat]))))
        List.cons(Nat.1, List.cons(Nat.1, List.cons(Nat.1, List.nil[Nat]))).is_unique
        unique_implies_tail_unique(Nat.1, List.cons(Nat.1, List.cons(Nat.1, List.nil[Nat])))
        List.cons(Nat.1, List.cons(Nat.1, List.nil[Nat])).is_unique
        unique_cons_head_not_in_tail(Nat.1, List.cons(Nat.1, List.nil[Nat]))
        List.cons(Nat.1, List.cons(Nat.1, List.nil[Nat])).is_unique implies not List.cons(Nat.1, List.nil[Nat]).contains(Nat.1)
        not List.cons(Nat.1, List.nil[Nat]).contains(Nat.1)
        cons_contains_head(Nat.1, List.nil[Nat])
        List.cons(Nat.1, List.nil[Nat]).contains(Nat.1)
        false
    }
    not List.cons(Nat.1, List.cons(Nat.1, List.cons(Nat.1, List.cons(Nat.1, List.nil[Nat])))).is_unique
}

/// The odd-part partitions of four: 3 + 1 and 1 + 1 + 1 + 1.
let odd_four_a: List[Nat] = List.cons(Nat.3, List.cons(Nat.1, List.nil[Nat]))
let odd_four_b: List[Nat] = List.cons(Nat.1, List.cons(Nat.1, List.cons(Nat.1, List.cons(Nat.1, List.nil[Nat]))))

/// The odd-part partitions of four are exactly 3 + 1 and 1 + 1 + 1 + 1.
theorem odd_four_members(l: List[Nat]) {
    is_odd_part_partition(l, Nat.4) implies (l = odd_four_a or l = odd_four_b)
} by {
    if is_odd_part_partition(l, Nat.4) {
        is_partition(l, Nat.4)
        all_odd(l)
        partition_four_char(l)
        is_partition(l, Nat.4) = (l = p4_a or l = p4_b or l = p4_c or l = p4_d or l = p4_e)
        l = p4_a or l = p4_b or l = p4_c or l = p4_d or l = p4_e
        p4_a = List.cons(Nat.4, List.nil[Nat])
        p4_b = List.cons(Nat.3, List.cons(Nat.1, List.nil[Nat]))
        p4_c = List.cons(Nat.2, List.cons(Nat.2, List.nil[Nat]))
        p4_d = List.cons(Nat.2, List.cons(Nat.1, List.cons(Nat.1, List.nil[Nat])))
        p4_e = List.cons(Nat.1, List.cons(Nat.1, List.cons(Nat.1, List.cons(Nat.1, List.nil[Nat]))))
        if l = p4_a {
            l = List.cons(Nat.4, List.nil[Nat])
            all_odd_4
            all_odd(List.cons(Nat.4, List.nil[Nat])) = false
            all_odd(l) = false
            all_odd(l)
            false
        }
        if not l = p4_a {
            if l = p4_b {
                l = List.cons(Nat.3, List.cons(Nat.1, List.nil[Nat]))
                l = odd_four_a
                or_intro_left(l = odd_four_a, l = odd_four_b)
                l = odd_four_a implies (l = odd_four_a or l = odd_four_b)
                l = odd_four_a or l = odd_four_b
            }
            if not l = p4_b {
                if l = p4_c {
                    l = List.cons(Nat.2, List.cons(Nat.2, List.nil[Nat]))
                    all_odd_22
                    all_odd(List.cons(Nat.2, List.cons(Nat.2, List.nil[Nat]))) = false
                    all_odd(l) = false
                    all_odd(l)
                    false
                }
                if not l = p4_c {
                    if l = p4_d {
                        l = List.cons(Nat.2, List.cons(Nat.1, List.cons(Nat.1, List.nil[Nat])))
                        all_odd_211
                        all_odd(List.cons(Nat.2, List.cons(Nat.1, List.cons(Nat.1, List.nil[Nat])))) = false
                        all_odd(l) = false
                        all_odd(l)
                        false
                    }
                    if not l = p4_d {
                        l = p4_e
                        l = List.cons(Nat.1, List.cons(Nat.1, List.cons(Nat.1, List.cons(Nat.1, List.nil[Nat]))))
                        l = odd_four_b
                        or_intro_right(l = odd_four_a, l = odd_four_b)
                        l = odd_four_b implies (l = odd_four_a or l = odd_four_b)
                        l = odd_four_a or l = odd_four_b
                    }
                    l = odd_four_a or l = odd_four_b
                }
                l = odd_four_a or l = odd_four_b
            }
            l = odd_four_a or l = odd_four_b
        }
        l = odd_four_a or l = odd_four_b
    }
}

/// The lists 3 + 1 and 1 + 1 + 1 + 1 are odd-part partitions of four.
theorem odd_four_sound(l: List[Nat]) {
    (l = odd_four_a or l = odd_four_b) implies is_odd_part_partition(l, Nat.4)
} by {
    if l = odd_four_a or l = odd_four_b {
        odd_four_a = List.cons(Nat.3, List.cons(Nat.1, List.nil[Nat]))
        odd_four_b = List.cons(Nat.1, List.cons(Nat.1, List.cons(Nat.1, List.cons(Nat.1, List.nil[Nat]))))
        if l = odd_four_a {
            l = List.cons(Nat.3, List.cons(Nat.1, List.nil[Nat]))
            l = p4_b
            l = p4_a or l = p4_b or l = p4_c or l = p4_d or l = p4_e
            p4_sound(l)
            (l = p4_a or l = p4_b or l = p4_c or l = p4_d or l = p4_e) implies is_partition(l, Nat.4)
            is_partition(l, Nat.4)
            all_odd_31
            all_odd(List.cons(Nat.3, List.cons(Nat.1, List.nil[Nat]))) = true
            all_odd(l) = true
            all_odd(l)
            is_partition(l, Nat.4) and all_odd(l)
            is_odd_part_partition(l, Nat.4)
        }
        if l = odd_four_b {
            l = List.cons(Nat.1, List.cons(Nat.1, List.cons(Nat.1, List.cons(Nat.1, List.nil[Nat]))))
            l = p4_e
            l = p4_a or l = p4_b or l = p4_c or l = p4_d or l = p4_e
            p4_sound(l)
            (l = p4_a or l = p4_b or l = p4_c or l = p4_d or l = p4_e) implies is_partition(l, Nat.4)
            is_partition(l, Nat.4)
            all_odd_1111
            all_odd(List.cons(Nat.1, List.cons(Nat.1, List.cons(Nat.1, List.cons(Nat.1, List.nil[Nat]))))) = true
            all_odd(l) = true
            all_odd(l)
            is_partition(l, Nat.4) and all_odd(l)
            is_odd_part_partition(l, Nat.4)
        }
        is_odd_part_partition(l, Nat.4)
    }
}

/// The odd-part partitions of four are exactly 3 + 1 and 1 + 1 + 1 + 1.
theorem odd_four_char(l: List[Nat]) {
    is_odd_part_partition(l, Nat.4) = (l = odd_four_a or l = odd_four_b)
} by {
    odd_four_members(l)
    is_odd_part_partition(l, Nat.4) implies (l = odd_four_a or l = odd_four_b)
    odd_four_sound(l)
    (l = odd_four_a or l = odd_four_b) implies is_odd_part_partition(l, Nat.4)
    if is_odd_part_partition(l, Nat.4) {
        odd_four_members(l)
        is_odd_part_partition(l, Nat.4) implies (l = odd_four_a or l = odd_four_b)
        l = odd_four_a or l = odd_four_b
        is_odd_part_partition(l, Nat.4) = (l = odd_four_a or l = odd_four_b)
    }
    if not is_odd_part_partition(l, Nat.4) {
        if l = odd_four_a or l = odd_four_b {
            odd_four_sound(l)
            (l = odd_four_a or l = odd_four_b) implies is_odd_part_partition(l, Nat.4)
            is_odd_part_partition(l, Nat.4)
            false
        }
        not (l = odd_four_a or l = odd_four_b)
        is_odd_part_partition(l, Nat.4) = (l = odd_four_a or l = odd_four_b)
    }
    is_odd_part_partition(l, Nat.4) = (l = odd_four_a or l = odd_four_b)
}

/// The distinct-part partitions of four are exactly 4 and 3 + 1.
theorem distinct_four_members(l: List[Nat]) {
    is_distinct_part_partition(l, Nat.4) implies (l = p4_a or l = p4_b)
} by {
    if is_distinct_part_partition(l, Nat.4) {
        is_partition(l, Nat.4)
        l.is_unique
        partition_four_char(l)
        is_partition(l, Nat.4) = (l = p4_a or l = p4_b or l = p4_c or l = p4_d or l = p4_e)
        l = p4_a or l = p4_b or l = p4_c or l = p4_d or l = p4_e
        if l = p4_a {
            or_intro_left(l = p4_a, l = p4_b)
            l = p4_a implies (l = p4_a or l = p4_b)
            l = p4_a or l = p4_b
        }
        if not l = p4_a {
            if l = p4_b {
                or_intro_right(l = p4_a, l = p4_b)
                l = p4_b implies (l = p4_a or l = p4_b)
                l = p4_a or l = p4_b
            }
            if not l = p4_b {
                if l = p4_c {
                    l = List.cons(Nat.2, List.cons(Nat.2, List.nil[Nat]))
                    not_unique_22
                    not List.cons(Nat.2, List.cons(Nat.2, List.nil[Nat])).is_unique
                    not l.is_unique
                    l.is_unique
                    false
                }
                if not l = p4_c {
                    if l = p4_d {
                        l = List.cons(Nat.2, List.cons(Nat.1, List.cons(Nat.1, List.nil[Nat])))
                        not_unique_211
                        not List.cons(Nat.2, List.cons(Nat.1, List.cons(Nat.1, List.nil[Nat]))).is_unique
                        not l.is_unique
                        l.is_unique
                        false
                    }
                    if not l = p4_d {
                        l = p4_e
                        l = List.cons(Nat.1, List.cons(Nat.1, List.cons(Nat.1, List.cons(Nat.1, List.nil[Nat]))))
                        not_unique_1111
                        not List.cons(Nat.1, List.cons(Nat.1, List.cons(Nat.1, List.cons(Nat.1, List.nil[Nat])))).is_unique
                        not l.is_unique
                        l.is_unique
                        false
                    }
                    l = p4_a or l = p4_b
                }
                l = p4_a or l = p4_b
            }
            l = p4_a or l = p4_b
        }
        l = p4_a or l = p4_b
    }
}

/// The lists 4 and 3 + 1 are distinct-part partitions of four.
theorem distinct_four_sound(l: List[Nat]) {
    (l = p4_a or l = p4_b) implies is_distinct_part_partition(l, Nat.4)
} by {
    if l = p4_a or l = p4_b {
        if l = p4_a {
            l = List.cons(Nat.4, List.nil[Nat])
            l = p4_a or l = p4_b or l = p4_c or l = p4_d or l = p4_e
            p4_sound(l)
            (l = p4_a or l = p4_b or l = p4_c or l = p4_d or l = p4_e) implies is_partition(l, Nat.4)
            is_partition(l, Nat.4)
            singleton_unique(Nat.4)
            List.singleton(Nat.4).is_unique
            List.cons(Nat.4, List.nil[Nat]) = List.singleton(Nat.4)
            List.cons(Nat.4, List.nil[Nat]).is_unique
            l.is_unique
            is_partition(l, Nat.4) and l.is_unique
            is_distinct_part_partition(l, Nat.4)
        }
        if l = p4_b {
            l = List.cons(Nat.3, List.cons(Nat.1, List.nil[Nat]))
            l = p4_a or l = p4_b or l = p4_c or l = p4_d or l = p4_e
            p4_sound(l)
            (l = p4_a or l = p4_b or l = p4_c or l = p4_d or l = p4_e) implies is_partition(l, Nat.4)
            is_partition(l, Nat.4)
            Nat.3 != Nat.1
            pair_nat_unique(Nat.3, Nat.1)
            Nat.3 != Nat.1 implies List.cons(Nat.3, List.cons(Nat.1, List.nil[Nat])).is_unique
            List.cons(Nat.3, List.cons(Nat.1, List.nil[Nat])).is_unique
            l.is_unique
            is_partition(l, Nat.4) and l.is_unique
            is_distinct_part_partition(l, Nat.4)
        }
        is_distinct_part_partition(l, Nat.4)
    }
}

/// The distinct-part partitions of four are exactly 4 and 3 + 1.
theorem distinct_four_char(l: List[Nat]) {
    is_distinct_part_partition(l, Nat.4) = (l = p4_a or l = p4_b)
} by {
    distinct_four_members(l)
    is_distinct_part_partition(l, Nat.4) implies (l = p4_a or l = p4_b)
    distinct_four_sound(l)
    (l = p4_a or l = p4_b) implies is_distinct_part_partition(l, Nat.4)
    if is_distinct_part_partition(l, Nat.4) {
        distinct_four_members(l)
        is_distinct_part_partition(l, Nat.4) implies (l = p4_a or l = p4_b)
        l = p4_a or l = p4_b
        is_distinct_part_partition(l, Nat.4) = (l = p4_a or l = p4_b)
    }
    if not is_distinct_part_partition(l, Nat.4) {
        if l = p4_a or l = p4_b {
            distinct_four_sound(l)
            (l = p4_a or l = p4_b) implies is_distinct_part_partition(l, Nat.4)
            is_distinct_part_partition(l, Nat.4)
            false
        }
        not (l = p4_a or l = p4_b)
        is_distinct_part_partition(l, Nat.4) = (l = p4_a or l = p4_b)
    }
    is_distinct_part_partition(l, Nat.4) = (l = p4_a or l = p4_b)
}

/// The explicit list of the odd-part partitions of four.
let odd_four_list: List[List[Nat]] =
    List.cons(odd_four_a, List.cons(odd_four_b, List.nil[List[Nat]]))

/// The explicit list of the distinct-part partitions of four.
let distinct_four_list: List[List[Nat]] =
    List.cons(p4_a, List.cons(p4_b, List.nil[List[Nat]]))

/// The odd-part partitions of four are exactly two in number.
theorem odd_four_cardinality {
    odd_partition_set(Nat.4).cardinality_is(Nat.2)
} by {
    contains_set_intro(odd_four_list, odd_partition_set(Nat.4))
    forall(l: List[Nat]) {
        if odd_partition_set(Nat.4).contains(l) {
            odd_partition_set_contains_iff(Nat.4, l)
            odd_partition_set(Nat.4).contains(l) = is_odd_part_partition(l, Nat.4)
            is_odd_part_partition(l, Nat.4)
            odd_four_members(l)
            is_odd_part_partition(l, Nat.4) implies (l = odd_four_a or l = odd_four_b)
            l = odd_four_a or l = odd_four_b
            odd_four_list = List.cons(odd_four_a, List.cons(odd_four_b, List.nil[List[Nat]]))
            if l = odd_four_a {
                cons_contains_head(odd_four_a, List.cons(odd_four_b, List.nil[List[Nat]]))
                List.cons(odd_four_a, List.cons(odd_four_b, List.nil[List[Nat]])).contains(odd_four_a)
                odd_four_list.contains(odd_four_a)
                odd_four_list.contains(l)
            }
            if not l = odd_four_a {
                l = odd_four_b
                cons_contains_head(odd_four_b, List.nil[List[Nat]])
                List.cons(odd_four_b, List.nil[List[Nat]]).contains(odd_four_b)
                cons_contains_of_tail_contains(odd_four_a, List.cons(odd_four_b, List.nil[List[Nat]]), odd_four_b)
                List.cons(odd_four_b, List.nil[List[Nat]]).contains(odd_four_b) implies List.cons(odd_four_a, List.cons(odd_four_b, List.nil[List[Nat]])).contains(odd_four_b)
                List.cons(odd_four_a, List.cons(odd_four_b, List.nil[List[Nat]])).contains(odd_four_b)
                odd_four_list.contains(odd_four_b)
                odd_four_list.contains(l)
            }
            odd_four_list.contains(l)
        }
    }
    forall(l: List[Nat]) { odd_partition_set(Nat.4).contains(l) implies odd_four_list.contains(l) }
    odd_four_list.contains_set(odd_partition_set(Nat.4))
    odd_four_sound(odd_four_a)
    (odd_four_a = odd_four_a or odd_four_a = odd_four_b) implies is_odd_part_partition(odd_four_a, Nat.4)
    odd_four_a = odd_four_a
    is_odd_part_partition(odd_four_a, Nat.4)
    odd_partition_set_contains_iff(Nat.4, odd_four_a)
    odd_partition_set(Nat.4).contains(odd_four_a) = is_odd_part_partition(odd_four_a, Nat.4)
    odd_partition_set(Nat.4).contains(odd_four_a)
    odd_four_sound(odd_four_b)
    (odd_four_b = odd_four_a or odd_four_b = odd_four_b) implies is_odd_part_partition(odd_four_b, Nat.4)
    odd_four_b = odd_four_b
    is_odd_part_partition(odd_four_b, Nat.4)
    odd_partition_set_contains_iff(Nat.4, odd_four_b)
    odd_partition_set(Nat.4).contains(odd_four_b) = is_odd_part_partition(odd_four_b, Nat.4)
    odd_partition_set(Nat.4).contains(odd_four_b)
    forall(x: List[Nat]) {
        if odd_four_list.contains(x) {
            cons_contains_eq(odd_four_a, List.cons(odd_four_b, List.nil[List[Nat]]), x)
            odd_four_list.contains(x) = (x = odd_four_a or List.cons(odd_four_b, List.nil[List[Nat]]).contains(x))
            cons_contains_eq(odd_four_b, List.nil[List[Nat]], x)
            List.cons(odd_four_b, List.nil[List[Nat]]).contains(x) = (x = odd_four_b or List.nil[List[Nat]].contains(x))
            nil_not_contains(x)
            x = odd_four_a or x = odd_four_b
            if x = odd_four_a {
                odd_partition_set(Nat.4).contains(odd_four_a)
                odd_partition_set(Nat.4).contains(x)
            }
            if not x = odd_four_a {
                x = odd_four_b
                odd_partition_set(Nat.4).contains(odd_four_b)
                odd_partition_set(Nat.4).contains(x)
            }
            odd_partition_set(Nat.4).contains(x)
        }
    }
    forall(x: List[Nat]) { odd_four_list.contains(x) implies odd_partition_set(Nat.4).contains(x) }
    filter_all_keep(odd_four_list, odd_partition_set(Nat.4).contains)
    (forall(x: List[Nat]) { odd_four_list.contains(x) implies odd_partition_set(Nat.4).contains(x) }) implies odd_four_list.filter(odd_partition_set(Nat.4).contains) = odd_four_list
    odd_four_list.filter(odd_partition_set(Nat.4).contains) = odd_four_list
    Nat.3 != Nat.1
    cons_neq_head(Nat.3, Nat.1, List.cons(Nat.1, List.nil[Nat]), List.cons(Nat.1, List.cons(Nat.1, List.cons(Nat.1, List.nil[Nat]))))
    List.cons(Nat.3, List.cons(Nat.1, List.nil[Nat])) != List.cons(Nat.1, List.cons(Nat.1, List.cons(Nat.1, List.cons(Nat.1, List.nil[Nat]))))
    odd_four_a = List.cons(Nat.3, List.cons(Nat.1, List.nil[Nat]))
    odd_four_b = List.cons(Nat.1, List.cons(Nat.1, List.cons(Nat.1, List.cons(Nat.1, List.nil[Nat]))))
    odd_four_a != odd_four_b
    pair_is_unique(odd_four_a, odd_four_b)
    odd_four_a != odd_four_b implies List.cons(odd_four_a, List.cons(odd_four_b, List.nil[List[Nat]])).is_unique
    List.cons(odd_four_a, List.cons(odd_four_b, List.nil[List[Nat]])).is_unique
    odd_four_list.is_unique
    odd_four_list.unique = odd_four_list
    odd_four_list.length = List.cons(odd_four_b, List.nil[List[Nat]]).length.suc
    List.cons(odd_four_b, List.nil[List[Nat]]).length = List.nil[List[Nat]].length.suc
    List.nil[List[Nat]].length = Nat.0
    List.cons(odd_four_b, List.nil[List[Nat]]).length = Nat.0.suc
    odd_four_list.length = Nat.0.suc.suc
    Nat.0.suc.suc = Nat.2
    odd_four_list.length = Nat.2
    odd_four_list.filter(odd_partition_set(Nat.4).contains).unique = odd_four_list
    odd_four_list.filter(odd_partition_set(Nat.4).contains).unique.length = Nat.2
    odd_four_list.contains_set(odd_partition_set(Nat.4)) and
        odd_four_list.filter(odd_partition_set(Nat.4).contains).unique.length = Nat.2
    exists(containing_list: List[List[Nat]]) {
        containing_list.contains_set(odd_partition_set(Nat.4)) and
            containing_list.filter(odd_partition_set(Nat.4).contains).unique.length = Nat.2
    }
    odd_partition_set(Nat.4).cardinality_is(Nat.2)
}

/// The distinct-part partitions of four are exactly two in number.
theorem distinct_four_cardinality {
    distinct_partition_set(Nat.4).cardinality_is(Nat.2)
} by {
    contains_set_intro(distinct_four_list, distinct_partition_set(Nat.4))
    forall(l: List[Nat]) {
        if distinct_partition_set(Nat.4).contains(l) {
            distinct_partition_set_contains_iff(Nat.4, l)
            distinct_partition_set(Nat.4).contains(l) = is_distinct_part_partition(l, Nat.4)
            is_distinct_part_partition(l, Nat.4)
            distinct_four_members(l)
            is_distinct_part_partition(l, Nat.4) implies (l = p4_a or l = p4_b)
            l = p4_a or l = p4_b
            distinct_four_list = List.cons(p4_a, List.cons(p4_b, List.nil[List[Nat]]))
            if l = p4_a {
                cons_contains_head(p4_a, List.cons(p4_b, List.nil[List[Nat]]))
                List.cons(p4_a, List.cons(p4_b, List.nil[List[Nat]])).contains(p4_a)
                distinct_four_list.contains(p4_a)
                distinct_four_list.contains(l)
            }
            if not l = p4_a {
                l = p4_b
                cons_contains_head(p4_b, List.nil[List[Nat]])
                List.cons(p4_b, List.nil[List[Nat]]).contains(p4_b)
                cons_contains_of_tail_contains(p4_a, List.cons(p4_b, List.nil[List[Nat]]), p4_b)
                List.cons(p4_b, List.nil[List[Nat]]).contains(p4_b) implies List.cons(p4_a, List.cons(p4_b, List.nil[List[Nat]])).contains(p4_b)
                List.cons(p4_a, List.cons(p4_b, List.nil[List[Nat]])).contains(p4_b)
                distinct_four_list.contains(p4_b)
                distinct_four_list.contains(l)
            }
            distinct_four_list.contains(l)
        }
    }
    forall(l: List[Nat]) { distinct_partition_set(Nat.4).contains(l) implies distinct_four_list.contains(l) }
    distinct_four_list.contains_set(distinct_partition_set(Nat.4))
    distinct_four_sound(p4_a)
    (p4_a = p4_a or p4_a = p4_b) implies is_distinct_part_partition(p4_a, Nat.4)
    p4_a = p4_a
    is_distinct_part_partition(p4_a, Nat.4)
    distinct_partition_set_contains_iff(Nat.4, p4_a)
    distinct_partition_set(Nat.4).contains(p4_a) = is_distinct_part_partition(p4_a, Nat.4)
    distinct_partition_set(Nat.4).contains(p4_a)
    distinct_four_sound(p4_b)
    (p4_b = p4_a or p4_b = p4_b) implies is_distinct_part_partition(p4_b, Nat.4)
    p4_b = p4_b
    is_distinct_part_partition(p4_b, Nat.4)
    distinct_partition_set_contains_iff(Nat.4, p4_b)
    distinct_partition_set(Nat.4).contains(p4_b) = is_distinct_part_partition(p4_b, Nat.4)
    distinct_partition_set(Nat.4).contains(p4_b)
    forall(x: List[Nat]) {
        if distinct_four_list.contains(x) {
            cons_contains_eq(p4_a, List.cons(p4_b, List.nil[List[Nat]]), x)
            distinct_four_list.contains(x) = (x = p4_a or List.cons(p4_b, List.nil[List[Nat]]).contains(x))
            cons_contains_eq(p4_b, List.nil[List[Nat]], x)
            List.cons(p4_b, List.nil[List[Nat]]).contains(x) = (x = p4_b or List.nil[List[Nat]].contains(x))
            nil_not_contains(x)
            x = p4_a or x = p4_b
            if x = p4_a {
                distinct_partition_set(Nat.4).contains(p4_a)
                distinct_partition_set(Nat.4).contains(x)
            }
            if not x = p4_a {
                x = p4_b
                distinct_partition_set(Nat.4).contains(p4_b)
                distinct_partition_set(Nat.4).contains(x)
            }
            distinct_partition_set(Nat.4).contains(x)
        }
    }
    forall(x: List[Nat]) { distinct_four_list.contains(x) implies distinct_partition_set(Nat.4).contains(x) }
    filter_all_keep(distinct_four_list, distinct_partition_set(Nat.4).contains)
    (forall(x: List[Nat]) { distinct_four_list.contains(x) implies distinct_partition_set(Nat.4).contains(x) }) implies distinct_four_list.filter(distinct_partition_set(Nat.4).contains) = distinct_four_list
    distinct_four_list.filter(distinct_partition_set(Nat.4).contains) = distinct_four_list
    p4_a != p4_b
    pair_is_unique(p4_a, p4_b)
    p4_a != p4_b implies List.cons(p4_a, List.cons(p4_b, List.nil[List[Nat]])).is_unique
    List.cons(p4_a, List.cons(p4_b, List.nil[List[Nat]])).is_unique
    distinct_four_list.is_unique
    distinct_four_list.unique = distinct_four_list
    distinct_four_list.length = List.cons(p4_b, List.nil[List[Nat]]).length.suc
    List.cons(p4_b, List.nil[List[Nat]]).length = List.nil[List[Nat]].length.suc
    List.nil[List[Nat]].length = Nat.0
    List.cons(p4_b, List.nil[List[Nat]]).length = Nat.0.suc
    distinct_four_list.length = Nat.0.suc.suc
    Nat.0.suc.suc = Nat.2
    distinct_four_list.length = Nat.2
    distinct_four_list.filter(distinct_partition_set(Nat.4).contains).unique = distinct_four_list
    distinct_four_list.filter(distinct_partition_set(Nat.4).contains).unique.length = Nat.2
    distinct_four_list.contains_set(distinct_partition_set(Nat.4)) and
        distinct_four_list.filter(distinct_partition_set(Nat.4).contains).unique.length = Nat.2
    exists(containing_list: List[List[Nat]]) {
        containing_list.contains_set(distinct_partition_set(Nat.4)) and
            containing_list.filter(distinct_partition_set(Nat.4).contains).unique.length = Nat.2
    }
    distinct_partition_set(Nat.4).cardinality_is(Nat.2)
}

/// Euler's identity for four: the number of partitions of four into odd parts
/// equals the number of partitions of four into distinct parts (both two).
theorem euler_odd_distinct_four {
    exists(k: Nat) {
        odd_partition_set(Nat.4).cardinality_is(k) and distinct_partition_set(Nat.4).cardinality_is(k)
    }
} by {
    odd_four_cardinality
    odd_partition_set(Nat.4).cardinality_is(Nat.2)
    distinct_four_cardinality
    distinct_partition_set(Nat.4).cardinality_is(Nat.2)
    odd_partition_set(Nat.4).cardinality_is(Nat.2) and distinct_partition_set(Nat.4).cardinality_is(Nat.2)
    exists(k: Nat) {
        odd_partition_set(Nat.4).cardinality_is(k) and distinct_partition_set(Nat.4).cardinality_is(k)
    }
}

// ---------------------------------------------------------------------------
// p grows.
// ---------------------------------------------------------------------------

/// p(2) > p(1): the partition function grows from one to two.
theorem p_grows_one_two {
    p(Nat.1) < p(Nat.2)
} by {
    p_one
    p(Nat.1) = Nat.1
    p_two
    p(Nat.2) = Nat.2
    lt_suc(Nat.1)
    Nat.1 < Nat.2
    p(Nat.1) < p(Nat.2)
}

/// p(3) > p(2): the partition function grows from two to three.
theorem p_grows_two_three {
    p(Nat.2) < p(Nat.3)
} by {
    p_two
    p(Nat.2) = Nat.2
    p_three
    p(Nat.3) = Nat.3
    lt_suc(Nat.2)
    Nat.2 < Nat.3
    p(Nat.2) < p(Nat.3)
}

/// p(4) > p(3): the partition function grows from three to four.
theorem p_grows_three_four {
    p(Nat.3) < p(Nat.4)
} by {
    p_three
    p(Nat.3) = Nat.3
    p_four
    p(Nat.4) = Nat.5
    lt_suc(Nat.3)
    Nat.3 < Nat.4
    lt_suc(Nat.4)
    Nat.4 < Nat.5
    lt_trans(Nat.3, Nat.4, Nat.5)
    Nat.3 < Nat.5
    p(Nat.3) < p(Nat.4)
}

/// The partition function grows through the small values:
/// p(1) < p(2) < p(3) < p(4).
theorem p_monotone_small {
    p(Nat.1) < p(Nat.2) and p(Nat.2) < p(Nat.3) and p(Nat.3) < p(Nat.4)
} by {
    p_grows_one_two
    p(Nat.1) < p(Nat.2)
    p_grows_two_three
    p(Nat.2) < p(Nat.3)
    p_grows_three_four
    p(Nat.3) < p(Nat.4)
    p(Nat.1) < p(Nat.2) and p(Nat.2) < p(Nat.3) and p(Nat.3) < p(Nat.4)
}

// The general monotonicity: p(n + 1) > p(n) for every n >= 1.  Prepending a
// part 1 to each partition of `n` gives an injective map into the partitions
// of `n + 1` whose image omits the singleton partition [n + 1]; turning this
// into a strict cardinality inequality needs the injection machinery of the
// finite-set library, so the statement is recorded here for later work.
//
// theorem p_grows(n: Nat) {
//     Nat.1 <= n implies p(n) < p(n.suc)
// }

// ---------------------------------------------------------------------------
// Euler's pentagonal number theorem.
// ---------------------------------------------------------------------------

/// Twice the generalized pentagonal number `g(k) = k(3k - 1) / 2`, stated
/// without division: `2 * g(k) = k * (3k - 1)`.  The pentagonal numbers are
/// the values for `k = 1, -1, 2, -2, ...`: 1, 2, 5, 7, 12, 15, ...
define pentagonal_twice(k: Nat) -> Nat {
    k * (Nat.3 * k - Nat.1)
}

// Euler's pentagonal number theorem (coefficient form).  For every n,
//
//     p(n) = sum over k != 0 of (-1)^(k-1) p(n - g(k)),  g(k) = k(3k-1)/2,
//
// where the terms whose index is negative are omitted.  The proof needs the
// generating-function identity (the product form of the partition generating
// function) together with the pentagonal product identity; both are recorded
// as statements in `combinatorics.partitions`.  Here the recurrence is
// verified for n = 4: the k = 1 and k = -1 terms are the only ones with a
// non-negative index (k = 2 and k = -2 give p(4 - 5) and p(4 - 7), both
// zero), so the recurrence reads p(4) = p(3) + p(2).

/// The pentagonal recurrence for n = 4: p(4) = p(3) + p(2).
theorem pentagonal_recurrence_four {
    p(Nat.4) = p(Nat.3) + p(Nat.2)
} by {
    p_four
    p(Nat.4) = Nat.5
    p_three
    p(Nat.3) = Nat.3
    p_two
    p(Nat.2) = Nat.2
    Nat.3 + Nat.2 = Nat.5
    p(Nat.3) + p(Nat.2) = Nat.5
    p(Nat.4) = p(Nat.3) + p(Nat.2)
}

/// The pentagonal recurrence for n = 4 in sign-sum form:
/// p(4) - p(3) - p(2) = 0.
theorem pentagonal_recurrence_four_diff {
    p(Nat.4) - p(Nat.3) - p(Nat.2) = Nat.0
} by {
    pentagonal_recurrence_four
    p(Nat.4) = p(Nat.3) + p(Nat.2)
    add_comm(p(Nat.3), p(Nat.2))
    p(Nat.3) + p(Nat.2) = p(Nat.2) + p(Nat.3)
    p(Nat.4) = p(Nat.2) + p(Nat.3)
    add_imp_sub(p(Nat.2), p(Nat.3), p(Nat.4))
    p(Nat.2) + p(Nat.3) = p(Nat.4) implies p(Nat.4) - p(Nat.3) = p(Nat.2)
    p(Nat.2) + p(Nat.3) = p(Nat.4)
    p(Nat.4) - p(Nat.3) = p(Nat.2)
    sub_self(p(Nat.2))
    p(Nat.2) - p(Nat.2) = Nat.0
    p(Nat.4) - p(Nat.3) - p(Nat.2) = Nat.0
}

// The general pentagonal recurrence, stated for later work:
//
// theorem pentagonal_recurrence(n: Nat) {
//     p(n) = ... sum over the k != 0 with k(3k-1)/2 <= n ...
// }
