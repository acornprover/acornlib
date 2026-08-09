/// Divisibility lemmas for the Erdős–Ginzburg–Ziv theorem.
///
/// The congruence and divisibility arithmetic needed by the small cases.
/// Kept separate so the host file, which only states the existence theorems,
/// is verified without the `number_theory` theorems in context (those slow
/// the proof search on the small closure goals below the per-step timeout).

from nat import Nat, mul_one_left, mul_one_right, divides_self, divides_mul,
    div_imp_mod, mod_of_zero, div_sub_mod, sub_zero, mul_suc_right, mul_comm
from number_theory import congr_mod_trans, congr_mod_add, mod_congr_mod_self, mod_lt
from list import List, map, sum, partial, partial_one, partial_split_last

numerals Nat

/// Divisibility by `d` follows from congruence to zero modulo `d`.
///
/// (The library's `number_theory.congruence` module is private to its package,
/// so the bridge is restated here; it needs only the natural-number theorems.)
theorem congr_mod_zero_of_divides(d: Nat, x: Nat) {
    d.divides(x) implies x.congr_mod(Nat.0, d)
} by {
    if d.divides(x) {
        div_imp_mod(x, d)
        x.mod(d) = Nat.0
        mod_of_zero(d)
        Nat.0.mod(d) = Nat.0
        x.mod(d) = Nat.0.mod(d)
        x.congr_mod(Nat.0, d)
    }
}

/// Congruence to zero modulo `d` gives divisibility by `d`.
theorem divides_of_congr_mod_zero(d: Nat, x: Nat) {
    x.congr_mod(Nat.0, d) implies d.divides(x)
} by {
    if x.congr_mod(Nat.0, d) {
        x.mod(d) = Nat.0.mod(d)
        mod_of_zero(d)
        Nat.0.mod(d) = Nat.0
        x.mod(d) = Nat.0
        div_sub_mod(x, d)
        d.divides(x - x.mod(d))
        x - x.mod(d) = x - Nat.0
        sub_zero(x)
        x - Nat.0 = x
        d.divides(x)
    }
}

/// One divides every natural number.
theorem one_divides(a: Nat) {
    Nat.1.divides(a)
} by {
    mul_one_left(a)
    Nat.1 * a = a
    exists(c: Nat) { Nat.1 * c = a }
    Nat.1.divides(a)
}

/// The sum of the values of `f` at the positions chosen by `w`, over `n` positions.
define egz_wsum(f: Nat -> Nat, w: Nat -> Nat, n: Nat) -> Nat {
    partial(function(j: Nat) { f(w(j)) }, n)
}

/// The sum over one chosen position is the value at that position.
theorem egz_wsum_one(f: Nat -> Nat, w: Nat -> Nat) {
    egz_wsum(f, w, Nat.1) = f(w(0))
} by {
    partial_one(function(j: Nat) { f(w(j)) })
    partial(function(j: Nat) { f(w(j)) }, Nat.1) = f(w(0))
}

/// The sum over two chosen positions is the sum of the two values.
theorem egz_wsum_two(f: Nat -> Nat, w: Nat -> Nat) {
    egz_wsum(f, w, Nat.2) = f(w(0)) + f(w(1))
} by {
    partial_split_last(function(j: Nat) { f(w(j)) }, Nat.1)
    partial(function(j: Nat) { f(w(j)) }, Nat.1.suc) =
        partial(function(j: Nat) { f(w(j)) }, Nat.1) + f(w(1))
    partial_one(function(j: Nat) { f(w(j)) })
    partial(function(j: Nat) { f(w(j)) }, Nat.1) = f(w(0))
    partial(function(j: Nat) { f(w(j)) }, Nat.2) = f(w(0)) + f(w(1))
}

/// The sum over three chosen positions is the sum of the three values.
theorem egz_wsum_three(f: Nat -> Nat, w: Nat -> Nat) {
    egz_wsum(f, w, Nat.3) = f(w(0)) + f(w(1)) + f(w(2))
} by {
    partial_split_last(function(j: Nat) { f(w(j)) }, Nat.2)
    partial(function(j: Nat) { f(w(j)) }, Nat.2.suc) =
        partial(function(j: Nat) { f(w(j)) }, Nat.2) + f(w(2))
    partial_split_last(function(j: Nat) { f(w(j)) }, Nat.1)
    partial(function(j: Nat) { f(w(j)) }, Nat.1.suc) =
        partial(function(j: Nat) { f(w(j)) }, Nat.1) + f(w(1))
    partial_one(function(j: Nat) { f(w(j)) })
    partial(function(j: Nat) { f(w(j)) }, Nat.1) = f(w(0))
    partial(function(j: Nat) { f(w(j)) }, Nat.2) = f(w(0)) + f(w(1))
    partial(function(j: Nat) { f(w(j)) }, Nat.3) = f(w(0)) + f(w(1)) + f(w(2))
}

/// The remainder modulo two is below two.
theorem mod_two_lt(x: Nat) {
    x.mod(2) < Nat.2
} by {
    mod_lt(x, Nat.2)
    Nat.2 != Nat.0
    x.mod(2) < Nat.2
}

/// The parity of the value of `f` at position `i`.
define parity_of(f: Nat -> Nat, i: Nat) -> Nat {
    f(i).mod(2)
}

/// Two numbers of the same parity have an even sum.
theorem even_sum_of_same_parity(x: Nat, y: Nat) {
    x.mod(2) = y.mod(2) implies Nat.2.divides(x + y)
} by {
    if x.mod(2) = y.mod(2) {
        x.congr_mod(y, 2)
        congr_mod_add(x, y, y, y, Nat.2)
        (x + y).congr_mod(y + y, Nat.2)
        mul_comm(Nat.2, y)
        Nat.2 * y = y * Nat.2
        mul_suc_right(y, Nat.1)
        y * Nat.1.suc = y * Nat.1 + y
        mul_one_right(y)
        y * Nat.1 = y
        y * Nat.2 = y + y
        Nat.2 * y = y + y
        divides_self(Nat.2)
        Nat.2.divides(Nat.2)
        divides_mul(Nat.2, y, Nat.2)
        Nat.2.divides(Nat.2 * y)
        congr_mod_zero_of_divides(Nat.2, Nat.2 * y)
        (Nat.2 * y).congr_mod(Nat.0, Nat.2)
        y + y = Nat.2 * y
        congr_mod_trans(x + y, y + y, Nat.0, Nat.2)
        (x + y).congr_mod(Nat.0, Nat.2)
        divides_of_congr_mod_zero(Nat.2, x + y)
        Nat.2.divides(x + y)
    }
}

/// Three numbers congruent to `v` modulo three sum to a multiple of three.
theorem three_divides_of_equal_residues(f: Nat -> Nat, a: Nat, b: Nat, c: Nat, v: Nat) {
    f(a).mod(3) = v and f(b).mod(3) = v and f(c).mod(3) = v implies
    Nat.3.divides(f(a) + f(b) + f(c))
} by {
    if f(a).mod(3) = v and f(b).mod(3) = v and f(c).mod(3) = v {
        f(a).congr_mod(v, 3)
        f(b).congr_mod(v, 3)
        f(c).congr_mod(v, 3)
        congr_mod_add(f(a), f(b), v, v, Nat.3)
        (f(a) + f(b)).congr_mod(v + v, Nat.3)
        congr_mod_add(f(a) + f(b), f(c), v + v, v, Nat.3)
        (f(a) + f(b) + f(c)).congr_mod(v + v + v, Nat.3)
        mul_suc_right(Nat.2, v)
        Nat.2 * v.suc = Nat.2 * v + Nat.2
        v + v + v = Nat.3 * v
        divides_self(Nat.3)
        Nat.3.divides(Nat.3)
        divides_mul(Nat.3, v, Nat.3)
        Nat.3.divides(Nat.3 * v)
        congr_mod_zero_of_divides(Nat.3, Nat.3 * v)
        (Nat.3 * v).congr_mod(Nat.0, Nat.3)
        v + v + v = Nat.3 * v
        congr_mod_trans(f(a) + f(b) + f(c), v + v + v, Nat.0, Nat.3)
        (f(a) + f(b) + f(c)).congr_mod(Nat.0, Nat.3)
        divides_of_congr_mod_zero(Nat.3, f(a) + f(b) + f(c))
        Nat.3.divides(f(a) + f(b) + f(c))
    }
}

/// Numbers of residues zero, one and two sum to a multiple of three.
theorem three_divides_of_all_residues(f: Nat -> Nat, a: Nat, b: Nat, c: Nat) {
    f(a).mod(3) = Nat.0 and f(b).mod(3) = Nat.1 and f(c).mod(3) = Nat.2 implies
    Nat.3.divides(f(a) + f(b) + f(c))
} by {
    if f(a).mod(3) = Nat.0 and f(b).mod(3) = Nat.1 and f(c).mod(3) = Nat.2 {
        f(a).congr_mod(Nat.0, 3)
        f(b).congr_mod(Nat.1, 3)
        f(c).congr_mod(Nat.2, 3)
        congr_mod_add(f(a), f(b), Nat.0, Nat.1, Nat.3)
        (f(a) + f(b)).congr_mod(Nat.0 + Nat.1, Nat.3)
        congr_mod_add(f(a) + f(b), f(c), Nat.0 + Nat.1, Nat.2, Nat.3)
        (f(a) + f(b) + f(c)).congr_mod(Nat.0 + Nat.1 + Nat.2, Nat.3)
        Nat.0 + Nat.1 + Nat.2 = Nat.3
        divides_self(Nat.3)
        Nat.3.divides(Nat.3)
        congr_mod_zero_of_divides(Nat.3, Nat.3)
        Nat.3.congr_mod(Nat.0, Nat.3)
        Nat.0 + Nat.1 + Nat.2 = Nat.3
        congr_mod_trans(f(a) + f(b) + f(c), Nat.0 + Nat.1 + Nat.2, Nat.0, Nat.3)
        (f(a) + f(b) + f(c)).congr_mod(Nat.0, Nat.3)
        divides_of_congr_mod_zero(Nat.3, f(a) + f(b) + f(c))
        Nat.3.divides(f(a) + f(b) + f(c))
    }
}
