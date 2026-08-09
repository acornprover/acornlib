/// Index-map machinery for the Erdős–Ginzburg–Ziv theorem.
///
/// A subsequence of `n` elements of a sequence presented as a function
/// `f: Nat -> Nat` is an index map `w` choosing `n` pairwise distinct
/// in-bounds positions, together with the sum of the values at those positions.
///
/// The index maps for the small cases are the `pick_*` functions, and the
/// bounds/injectivity facts about them are proved here with only the natural
/// numbers and lists in context, so the host file can cite them.

from nat import Nat, not_lt_zero, lt_suc_right, lt_imp_lte_suc, lte_cancel_suc,
    only_zero_lte_zero, lt_and_lte, lt_suc, lt_trans, lt_not_ref
from list import List, map, sum, partial

numerals Nat

/// The index map picking the single position `i`.
define pick_one(i: Nat) -> Nat -> Nat {
    function(a: Nat) {
        i
    }
}

/// The entry of `pick_one` is `i`.
theorem pick_one_apply(i: Nat, a: Nat) {
    pick_one(i)(a) = i
}

/// The index map picking the two positions `i` and `j`.
define pick_two(i: Nat, j: Nat) -> Nat -> Nat {
    function(a: Nat) {
        if a = 0 {
            i
        } else {
            j
        }
    }
}

/// The first entry of `pick_two` is `i`.
theorem pick_two_zero(i: Nat, j: Nat) {
    pick_two(i, j)(0) = i
}

/// The second entry of `pick_two` is `j`.
theorem pick_two_one(i: Nat, j: Nat) {
    pick_two(i, j)(1) = j
} by {
    Nat.1 != Nat.0
    pick_two(i, j)(1) = j
}

/// The index map picking the three positions `i`, `j` and `k`.
define pick_three(i: Nat, j: Nat, k: Nat) -> Nat -> Nat {
    function(a: Nat) {
        if a = 0 {
            i
        } else {
            if a = 1 {
                j
            } else {
                k
            }
        }
    }
}

/// The first entry of `pick_three` is `i`.
theorem pick_three_zero(i: Nat, j: Nat, k: Nat) {
    pick_three(i, j, k)(0) = i
}

/// The second entry of `pick_three` is `j`.
theorem pick_three_one(i: Nat, j: Nat, k: Nat) {
    pick_three(i, j, k)(1) = j
} by {
    Nat.1 != Nat.0
    pick_three(i, j, k)(1) = j
}

/// The third entry of `pick_three` is `k`.
theorem pick_three_two(i: Nat, j: Nat, k: Nat) {
    pick_three(i, j, k)(2) = k
} by {
    Nat.2 != Nat.0
    Nat.2 != Nat.1
    pick_three(i, j, k)(2) = k
}

/// The single position zero is below one.
theorem egz_bounds_one {
    forall(j: Nat) { j < Nat.1 implies pick_one(Nat.0)(j) < Nat.1 }
} by {
    forall(j: Nat) {
        if j < Nat.1 {
            lt_suc_right(j, Nat.0)
            j = Nat.0 or j < Nat.0
            if j = Nat.0 {
                pick_one_apply(Nat.0, j)
                pick_one(Nat.0)(j) = Nat.0
                lt_suc(Nat.0)
                Nat.0 < Nat.1
                pick_one(Nat.0)(j) < Nat.1
            }
            if j < Nat.0 {
                not_lt_zero(j)
                false
            }
            pick_one(Nat.0)(j) < Nat.1
        }
    }
}

/// The single position is vacuously distinct for `n = 1`.
theorem egz_inj_one {
    forall(a: Nat, b: Nat) { a < b and b < Nat.1 implies pick_one(Nat.0)(a) != pick_one(Nat.0)(b) }
} by {
    forall(a: Nat, b: Nat) {
        if a < b and b < Nat.1 {
            lt_suc_right(b, Nat.0)
            b = Nat.0 or b < Nat.0
            if b = Nat.0 {
                a < Nat.0
                not_lt_zero(a)
                false
            }
            if b < Nat.0 {
                not_lt_zero(b)
                false
            }
            false
        }
    }
}

/// Two positions `i < j < 3`: the picked positions are below three.
theorem egz_bounds_two(i: Nat, j: Nat) {
    i < j and j < Nat.3 implies forall(j2: Nat) {
        j2 < Nat.2 implies pick_two(i, j)(j2) < Nat.3
    }
} by {
    if i < j and j < Nat.3 {
        lt_trans(i, j, Nat.3)
        i < Nat.3
        forall(j2: Nat) {
            if j2 < Nat.2 {
                lt_suc_right(j2, Nat.1)
                j2 = Nat.1 or j2 < Nat.1
                if j2 = Nat.1 {
                    pick_two_one(i, j)
                    pick_two(i, j)(1) = j
                    j < Nat.3
                    pick_two(i, j)(j2) < Nat.3
                }
                if j2 < Nat.1 {
                    lt_suc_right(j2, Nat.0)
                    j2 = Nat.0 or j2 < Nat.0
                    if j2 = Nat.0 {
                        pick_two_zero(i, j)
                        pick_two(i, j)(0) = i
                        i < Nat.3
                        pick_two(i, j)(j2) < Nat.3
                    }
                    if j2 < Nat.0 {
                        not_lt_zero(j2)
                        false
                    }
                    pick_two(i, j)(j2) < Nat.3
                }
                pick_two(i, j)(j2) < Nat.3
            }
        }
    }
}

/// Two distinct positions `i < j < 3`: the picked positions are distinct.
/// Strict inequality implies inequality.
theorem lt_imp_ne_nat(a: Nat, b: Nat) {
    a < b implies a != b
} by {
    if a < b {
        if a = b {
            a < a
            lt_not_ref(a)
            false
        }
        a != b
    }
}

/// Two distinct positions `i < j < 3`: the picked positions are distinct.
theorem egz_inj_two(i: Nat, j: Nat) {
    i < j and j < Nat.3 implies forall(a: Nat, b: Nat) {
        a < b and b < Nat.2 implies pick_two(i, j)(a) != pick_two(i, j)(b)
    }
} by {
    if i < j and j < Nat.3 {
        lt_imp_ne_nat(i, j)
        i != j
        forall(a: Nat, b: Nat) {
            if a < b and b < Nat.2 {
                lt_suc_right(b, Nat.1)
                b = Nat.1 or b < Nat.1
                if b = Nat.1 {
                    lt_and_lte(a, b, Nat.1)
                    a < Nat.1
                    lt_imp_lte_suc(a, Nat.0)
                    a.suc <= Nat.1
                    lte_cancel_suc(a, Nat.0)
                    a <= Nat.0
                    only_zero_lte_zero(a)
                    a = Nat.0
                    pick_two_zero(i, j)
                    pick_two(i, j)(a) = i
                    pick_two_one(i, j)
                    pick_two(i, j)(b) = j
                    i != j
                    pick_two(i, j)(a) != pick_two(i, j)(b)
                }
                if b < Nat.1 {
                    lt_imp_lte_suc(b, Nat.0)
                    b.suc <= Nat.1
                    lte_cancel_suc(b, Nat.0)
                    b <= Nat.0
                    only_zero_lte_zero(b)
                    b = Nat.0
                    a < Nat.0
                    not_lt_zero(a)
                    false
                }
                pick_two(i, j)(a) != pick_two(i, j)(b)
            }
        }
    }
}

/// A number below two is zero or one.
theorem lt_two_cases(x: Nat) {
    x < Nat.2 implies x = Nat.0 or x = Nat.1
} by {
    if x < Nat.2 {
        lt_suc_right(x, Nat.1)
        x = Nat.1 or x < Nat.1
        if x = Nat.1 {
            x = Nat.0 or x = Nat.1
        }
        if x < Nat.1 {
            lt_suc_right(x, Nat.0)
            x = Nat.0 or x < Nat.0
            if x < Nat.0 {
                not_lt_zero(x)
                false
            }
            x = Nat.0 or x = Nat.1
        }
        x = Nat.0 or x = Nat.1
    }
}

/// A nonzero number below two is one.
theorem lt_two_ne_zero_imp_one(x: Nat) {
    x < Nat.2 and x != Nat.0 implies x = Nat.1
} by {
    if x < Nat.2 and x != Nat.0 {
        lt_two_cases(x)
        x = Nat.0 or x = Nat.1
        if x = Nat.0 {
            x != Nat.0
            false
        }
        x = Nat.1
    }
}

/// A non-one number below two is zero.
theorem lt_two_ne_one_imp_zero(x: Nat) {
    x < Nat.2 and x != Nat.1 implies x = Nat.0
} by {
    if x < Nat.2 and x != Nat.1 {
        lt_two_cases(x)
        x = Nat.0 or x = Nat.1
        if x = Nat.1 {
            x != Nat.1
            false
        }
        x = Nat.0
    }
}
