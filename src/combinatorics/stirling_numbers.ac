/// The Stirling numbers of the second kind.
///
/// The Stirling number `S(n, k)` of the second kind counts the number of
/// partitions of an `n`-element set into `k` nonempty blocks.  They satisfy
/// the classical recurrence
///
///     S(n + 1, k + 1) = S(n, k) + (k + 1) * S(n, k + 1),
///
/// with `S(0, 0) = 1` and `S(n, 0) = 0` for `n > 0`, which is used here as
/// the definition.
///
/// The file proves the recurrence, the boundary values `S(n, 1) = 1`,
/// `S(n, n) = 1`, `S(n, k) = 0` for `k > n`, the closed form
/// `S(n, 2) = 2^(n - 1) - 1`, and the classical identity connecting the
/// Stirling numbers to falling factorials:
///
///     n^m = sum_{k=0}^{m-1} S(m, k + 1) * falling_product(n, k),
///
/// where `falling_product(n, k) = n * (n - 1) * ... * (n - k)` is the
/// library's falling product of `k + 1` factors.
///
/// (Stirling's *approximation* `n! ~ sqrt(2 pi n) (n / e)^n`, which is the
/// content of `theorems1000/theorem_stirling.ac`, is a separate asymptotic
/// statement of analysis and is not treated here.)

from nat import Nat, add_sub, sub_self, lt_imp_lte_suc, lte_trans,
    mul_suc_right, mul_comm, add_comm, alt_induction, lt_imp_lt_suc,
    lt_or_lte, lt_trans, add_imp_sub_left, suc_sub_one,
    lte_add_right, lte_ref, pow_one, pow_add, not_lt_zero, lt_cancel_suc,
    lte_mul, add_cancels_right, only_zero_lte_zero, distrib_right, lte_antisymm,
    mul_assoc, distrib_left
from list import partial, partial_split_last, partial_pointwise_eq,
    partial_scalar_mul, partial_add, partial_shift_suc
from data.basic.functions import compose
from algebra.semigroup import mul_fn
from algebra.add_semigroup import add_fn
from number_theory import falling_product, falling_product_suc, falling_product_zero
from combinatorics.catalan import nat_mul_sub_distrib_left, nat_add_sub_cancel,
    nat_mul_sub_distrib_right, nat_pred_add_one

numerals Nat

/// The Stirling number of the second kind `S(n, k)`: the number of partitions
/// of an `n`-element set into `k` nonempty blocks.
define stirling(n: Nat, k: Nat) -> Nat {
    match n {
        Nat.zero {
            match k {
                Nat.zero {
                    Nat.1
                }
                Nat.suc(_) {
                    Nat.0
                }
            }
        }
        Nat.suc(m) {
            match k {
                Nat.zero {
                    Nat.0
                }
                Nat.suc(j) {
                    stirling(m, j) + k * stirling(m, k)
                }
            }
        }
    }
}

/// The induction conclusion of `stirling_one` instantiated at n.
lemma stirling_one_all(n: Nat) {
    (forall(m: Nat) {
        Nat.0 < m implies stirling(m, Nat.1) = Nat.1
    }) implies (Nat.0 < n implies stirling(n, Nat.1) = Nat.1)
} by {
    if forall(m: Nat) {
        Nat.0 < m implies stirling(m, Nat.1) = Nat.1
    } {
        if Nat.0 < n {
            Nat.0 < n implies stirling(n, Nat.1) = Nat.1
            stirling(n, Nat.1) = Nat.1
        }
    }
    if not forall(m: Nat) {
        Nat.0 < m implies stirling(m, Nat.1) = Nat.1
    } {
        false
    }
}

/// The induction conclusion of `stirling_out_of_bounds` instantiated at n.
lemma stirling_out_all(n: Nat) {
    (forall(m: Nat, k2: Nat) {
        m < k2 implies stirling(m, k2) = Nat.0
    }) implies (forall(k2: Nat) {
        n < k2 implies stirling(n, k2) = Nat.0
    })
} by {
    if forall(m: Nat, k2: Nat) {
        m < k2 implies stirling(m, k2) = Nat.0
    } {
        forall(k2: Nat) {
            n < k2 implies stirling(n, k2) = Nat.0
        }
    }
    if not forall(m: Nat, k2: Nat) {
        m < k2 implies stirling(m, k2) = Nat.0
    } {
        false
    }
}

/// The induction conclusion of `stirling_self` instantiated at n.
lemma stirling_self_all(n: Nat) {
    (forall(m: Nat) {
        stirling(m, m) = Nat.1
    }) implies stirling(n, n) = Nat.1
} by {
    if forall(m: Nat) {
        stirling(m, m) = Nat.1
    } {
        stirling(n, n) = Nat.1
    }
    if not forall(m: Nat) {
        stirling(m, m) = Nat.1
    } {
        false
    }
}

/// The induction conclusion of `stirling_two` instantiated at n.
lemma stirling_two_all(n: Nat) {
    (forall(m: Nat) {
        Nat.0 < m implies stirling(m, Nat.2) = Nat.2.pow(m - Nat.1) - Nat.1
    }) implies (Nat.0 < n implies stirling(n, Nat.2) = Nat.2.pow(n - Nat.1) - Nat.1)
} by {
    if forall(m: Nat) {
        Nat.0 < m implies stirling(m, Nat.2) = Nat.2.pow(m - Nat.1) - Nat.1
    } {
        if Nat.0 < n {
            Nat.0 < n implies stirling(n, Nat.2) = Nat.2.pow(n - Nat.1) - Nat.1
            stirling(n, Nat.2) = Nat.2.pow(n - Nat.1) - Nat.1
        }
    }
    if not forall(m: Nat) {
        Nat.0 < m implies stirling(m, Nat.2) = Nat.2.pow(m - Nat.1) - Nat.1
    } {
        false
    }
}

/// S(0, 0) = 1.
theorem stirling_zero_zero {
    stirling(Nat.0, Nat.0) = Nat.1
} by {
    stirling(Nat.0, Nat.0) = Nat.1
}

/// The recurrence: S(n + 1, k + 1) = S(n, k) + (k + 1) * S(n, k + 1).
theorem stirling_recurrence(n: Nat, k: Nat) {
    stirling(n.suc, k.suc) = stirling(n, k) + k.suc * stirling(n, k.suc)
} by {
    stirling(n.suc, k.suc) = stirling(n, k) + k.suc * stirling(n, k.suc)
}

/// S(n, 0) = 0 for every positive n.
theorem stirling_zero_col(n: Nat) {
    Nat.0 < n implies stirling(n, Nat.0) = Nat.0
} by {
    if Nat.0 < n {
        let m: Nat satisfy {
            m.suc = n
        }
        stirling(m.suc, Nat.0) = Nat.0
        stirling(n, Nat.0) = Nat.0
    }
}

/// S(1, 1) = 1.
theorem stirling_one_one {
    stirling(Nat.1, Nat.1) = Nat.1
} by {
    stirling(Nat.1, Nat.1) = stirling(Nat.0, Nat.0) + Nat.1 * stirling(Nat.0, Nat.1)
    stirling(Nat.0, Nat.0) = Nat.1
    stirling(Nat.0, Nat.1) = Nat.0
    Nat.1 * Nat.0 = Nat.0
    stirling(Nat.1, Nat.1) = Nat.1 + Nat.0
    Nat.1 + Nat.0 = Nat.1
    stirling(Nat.1, Nat.1) = Nat.1
}

/// S(n, 1) = 1 for every positive n.
theorem stirling_one(n: Nat) {
    Nat.0 < n implies stirling(n, Nat.1) = Nat.1
} by {
    define p(m: Nat) -> Bool {
        Nat.0 < m implies stirling(m, Nat.1) = Nat.1
    }
    // p(0) is vacuous
    if Nat.0 < Nat.0 {
        not_lt_zero(Nat.0)
        not (Nat.0 < Nat.0)
        false
    }
    p(Nat.0)
    forall(m: Nat) {
        if p(m) {
            if Nat.0 < m.suc {
                stirling(m.suc, Nat.1) =
                    stirling(m, Nat.0) + Nat.1 * stirling(m, Nat.1)
                if Nat.0 < m {
                    stirling_zero_col(m)
                    stirling(m, Nat.0) = Nat.0
                    p(m) = (Nat.0 < m implies stirling(m, Nat.1) = Nat.1)
                    stirling(m, Nat.1) = Nat.1
                    Nat.1 * stirling(m, Nat.1) = stirling(m, Nat.1)
                    stirling(m.suc, Nat.1) = Nat.0 + stirling(m, Nat.1)
                    Nat.0 + stirling(m, Nat.1) = stirling(m, Nat.1)
                    stirling(m.suc, Nat.1) = Nat.1
                } else {
                    Nat.0 = m
                    stirling_one_one
                    stirling(Nat.1, Nat.1) = Nat.1
                    stirling(m.suc, Nat.1) = stirling(Nat.1, Nat.1)
                    stirling(m.suc, Nat.1) = Nat.1
                }
            }
            p(m.suc)
        }
    }
    alt_induction(p)
    p(Nat.0) and forall(m: Nat) { p(m) implies p(m.suc) }
    forall(m: Nat) { p(m) }
    if forall(m: Nat) { p(m) } {
        forall(m: Nat) {
            p(m) = (Nat.0 < m implies stirling(m, Nat.1) = Nat.1)
            Nat.0 < m implies stirling(m, Nat.1) = Nat.1
        }
        stirling_one_all(n)
        if Nat.0 < n {
            Nat.0 < n implies stirling(n, Nat.1) = Nat.1
            stirling(n, Nat.1) = Nat.1
        }
    }
    if not forall(m: Nat) { p(m) } {
        false
    }
}


/// S(n, k) = 0 whenever k > n.
theorem stirling_out_of_bounds(n: Nat, k: Nat) {
    n < k implies stirling(n, k) = Nat.0
} by {
    define p(m: Nat) -> Bool {
        forall(k2: Nat) {
            m < k2 implies stirling(m, k2) = Nat.0
        }
    }
    // base m = 0
    forall(k2: Nat) {
        if Nat.0 < k2 {
            let j2: Nat satisfy {
                j2.suc = k2
            }
            stirling(Nat.0, j2.suc) = Nat.0
            stirling(Nat.0, k2) = Nat.0
        }
    }
    p(Nat.0)
    // step
    forall(m: Nat) {
        if p(m) {
            forall(k2: Nat) {
                if m.suc < k2 {
                    if k2 = Nat.0 {
                        stirling(m.suc, Nat.0) = Nat.0
                        stirling(m.suc, k2) = Nat.0
                    } else {
                        let j: Nat satisfy {
                            j.suc = k2
                        }
                        m.suc < j.suc
                        lt_cancel_suc(m, j)
                        m < j
                        if forall(k3: Nat) {
                            m < k3 implies stirling(m, k3) = Nat.0
                        } {
                            m < j implies stirling(m, j) = Nat.0
                            stirling(m, j) = Nat.0
                            lt_imp_lt_suc(m, j)
                            m < j.suc
                            m < j.suc implies stirling(m, j.suc) = Nat.0
                            stirling(m, j.suc) = Nat.0
                            stirling(m.suc, j.suc) =
                                stirling(m, j) + j.suc * stirling(m, j.suc)
                            stirling(m.suc, j.suc) = Nat.0 + j.suc * Nat.0
                            j.suc * Nat.0 = Nat.0
                            stirling(m.suc, j.suc) = Nat.0
                            stirling(m.suc, k2) = Nat.0
                        }
                        if not forall(k3: Nat) {
                            m < k3 implies stirling(m, k3) = Nat.0
                        } {
                            false
                        }
                        stirling(m.suc, k2) = Nat.0
                    }
                }
            }
            p(m.suc)
        }
    }
    alt_induction(p)
    p(Nat.0) and forall(m: Nat) { p(m) implies p(m.suc) }
    forall(m: Nat) { p(m) }
    if forall(m: Nat) { p(m) } {
        forall(m: Nat) {
            p(m) = (forall(k2: Nat) {
                m < k2 implies stirling(m, k2) = Nat.0
            })
            forall(k2: Nat) {
                m < k2 implies stirling(m, k2) = Nat.0
            }
        }
        stirling_out_all(n)
        if n < k {
            n < k implies stirling(n, k) = Nat.0
            stirling(n, k) = Nat.0
        }
    }
    if not forall(m: Nat) { p(m) } {
        false
    }
}


/// S(n, n) = 1.
theorem stirling_self(n: Nat) {
    stirling(n, n) = Nat.1
} by {
    define p(m: Nat) -> Bool {
        stirling(m, m) = Nat.1
    }
    stirling_zero_zero
    stirling(Nat.0, Nat.0) = Nat.1
    p(Nat.0)
    forall(m: Nat) {
        if p(m) {
            stirling(m.suc, m.suc) =
                stirling(m, m) + m.suc * stirling(m, m.suc)
            p(m) = (stirling(m, m) = Nat.1)
            stirling(m, m) = Nat.1
            m < m.suc
            stirling_out_of_bounds(m, m.suc)
            stirling(m, m.suc) = Nat.0
            m.suc * Nat.0 = Nat.0
            stirling(m.suc, m.suc) = Nat.1 + Nat.0
            Nat.1 + Nat.0 = Nat.1
            stirling(m.suc, m.suc) = Nat.1
            p(m.suc)
        }
    }
    alt_induction(p)
    p(Nat.0) and forall(m: Nat) { p(m) implies p(m.suc) }
    forall(m: Nat) { p(m) }
    if forall(m: Nat) { p(m) } {
        forall(m: Nat) {
            p(m) = (stirling(m, m) = Nat.1)
            stirling(m, m) = Nat.1
        }
        stirling_self_all(n)
        stirling(n, n) = Nat.1
    }
    if not forall(m: Nat) { p(m) } {
        false
    }
}


/// 2 is at most 2^m for positive m.
theorem nat_two_lte_two(m: Nat) {
    Nat.0 < m implies Nat.2 <= Nat.2.pow(m)
} by {
    if Nat.0 < m {
        let j: Nat satisfy {
            j.suc = m
        }
        pow_add(Nat.2, Nat.1, j)
        Nat.2.pow(Nat.1) * Nat.2.pow(j) = Nat.2.pow(Nat.1 + j)
        pow_one[Nat](Nat.2)
        Nat.2.pow(Nat.1) = Nat.2
        Nat.1 + j = m
        Nat.2 * Nat.2.pow(j) = Nat.2.pow(m)
        lte_mul(Nat.2, Nat.2.pow(j))
        Nat.2.pow(j) != Nat.0
        Nat.2 <= Nat.2 * Nat.2.pow(j)
        Nat.2 <= Nat.2.pow(m)
    }
}

/// (a - 2) + 1 = a - 1 whenever a >= 2.
theorem nat_sub_two_add_one(a: Nat) {
    Nat.2 <= a implies (a - Nat.2) + Nat.1 = a - Nat.1
} by {
    if Nat.2 <= a {
        add_sub(a, Nat.2)
        a - Nat.2 + Nat.2 = a
        ((a - Nat.2) + Nat.1) + Nat.1 = (a - Nat.2) + Nat.2
        ((a - Nat.2) + Nat.1) + Nat.1 = a
        lte_trans(Nat.1, Nat.2, a)
        Nat.1 <= a
        add_sub(a, Nat.1)
        a - Nat.1 + Nat.1 = a
        add_cancels_right((a - Nat.2) + Nat.1, a - Nat.1, Nat.1)
        (a - Nat.2) + Nat.1 = a - Nat.1
    }
}

/// 2 * 2^(n - 1) = 2^n for positive n.
theorem nat_two_pow_pred(n: Nat) {
    Nat.0 < n implies Nat.2 * Nat.2.pow(n - Nat.1) = Nat.2.pow(n)
} by {
    if Nat.0 < n {
        pow_one[Nat](Nat.2)
        Nat.2.pow(Nat.1) = Nat.2
        pow_add(Nat.2, Nat.1, n - Nat.1)
        Nat.2.pow(Nat.1) * Nat.2.pow(n - Nat.1) = Nat.2.pow(Nat.1 + (n - Nat.1))
        Nat.2 * Nat.2.pow(n - Nat.1) = Nat.2.pow(Nat.1 + (n - Nat.1))
        nat_pred_add_one(n)
        n - Nat.1 + Nat.1 = n
        Nat.1 + (n - Nat.1) = n
        Nat.2 * Nat.2.pow(n - Nat.1) = Nat.2.pow(n)
    }
}

/// S(n, 2) = 2^(n - 1) - 1 for every positive n.
theorem stirling_two(n: Nat) {
    Nat.0 < n implies stirling(n, Nat.2) = Nat.2.pow(n - Nat.1) - Nat.1
} by {
    define p(m: Nat) -> Bool {
        Nat.0 < m implies stirling(m, Nat.2) = Nat.2.pow(m - Nat.1) - Nat.1
    }
    // base m = 1
    stirling(Nat.1, Nat.2) = stirling(Nat.0, Nat.1) + Nat.2 * stirling(Nat.0, Nat.2)
    stirling(Nat.0, Nat.1) = Nat.0
    stirling(Nat.0, Nat.2) = Nat.0
    Nat.2 * Nat.0 = Nat.0
    stirling(Nat.1, Nat.2) = Nat.0
    Nat.1 - Nat.1 = Nat.0
    Nat.2.pow(Nat.0) = Nat.1
    Nat.2.pow(Nat.1 - Nat.1) - Nat.1 = Nat.1 - Nat.1
    Nat.1 - Nat.1 = Nat.0
    Nat.2.pow(Nat.1 - Nat.1) - Nat.1 = Nat.0
    stirling(Nat.1, Nat.2) = Nat.2.pow(Nat.1 - Nat.1) - Nat.1
    p(Nat.1)
    // p(0) is vacuous
    if Nat.0 < Nat.0 {
        not_lt_zero(Nat.0)
        not (Nat.0 < Nat.0)
        false
    }
    p(Nat.0)
    // step
    forall(m: Nat) {
        if p(m) {
            if Nat.0 < m.suc {
                stirling(m.suc, Nat.2) =
                    stirling(m, Nat.1) + Nat.2 * stirling(m, Nat.2)
                if Nat.0 < m {
                    stirling_one(m)
                    stirling(m, Nat.1) = Nat.1
                    p(m)
                    stirling(m, Nat.2) = Nat.2.pow(m - Nat.1) - Nat.1
                    nat_mul_sub_distrib_left(Nat.2, Nat.2.pow(m - Nat.1), Nat.1)
                    Nat.2 * (Nat.2.pow(m - Nat.1) - Nat.1) =
                        Nat.2 * Nat.2.pow(m - Nat.1) - Nat.2 * Nat.1
                    Nat.2 * Nat.1 = Nat.2
                    Nat.2 * (Nat.2.pow(m - Nat.1) - Nat.1) =
                        Nat.2 * Nat.2.pow(m - Nat.1) - Nat.2
                    nat_two_pow_pred(m)
                    Nat.2 * Nat.2.pow(m - Nat.1) = Nat.2.pow(m)
                    Nat.2 * (Nat.2.pow(m - Nat.1) - Nat.1) =
                        Nat.2.pow(m) - Nat.2
                    stirling(m.suc, Nat.2) = Nat.1 + (Nat.2.pow(m) - Nat.2)
                    // 1 + (2^m - 2) = 2^m - 1
                    nat_two_lte_two(m)
                    Nat.2 <= Nat.2.pow(m)
                    nat_sub_two_add_one(Nat.2.pow(m))
                    (Nat.2.pow(m) - Nat.2) + Nat.1 = Nat.2.pow(m) - Nat.1
                    add_comm(Nat.1, Nat.2.pow(m) - Nat.2)
                    Nat.1 + (Nat.2.pow(m) - Nat.2) =
                        Nat.2.pow(m) - Nat.2 + Nat.1
                    stirling(m.suc, Nat.2) = Nat.2.pow(m) - Nat.1
                    nat_two_pow_pred(m)
                    Nat.2 * Nat.2.pow(m - Nat.1) = Nat.2.pow(m)
                    Nat.2.pow(m) - Nat.1 = Nat.2 * Nat.2.pow(m - Nat.1) - Nat.1
                    m - Nat.1 + Nat.1 = m
                    Nat.2.pow(m.suc - Nat.1) - Nat.1 =
                        Nat.2.pow(m) - Nat.1
                    stirling(m.suc, Nat.2) =
                        Nat.2.pow(m.suc - Nat.1) - Nat.1
                } else {
                    Nat.0 = m
                    stirling(Nat.1, Nat.2) = Nat.0
                    Nat.1.suc - Nat.1 = Nat.1
                    Nat.2.pow(Nat.1.suc - Nat.1) - Nat.1 =
                        Nat.2.pow(Nat.1) - Nat.1
                    Nat.2.pow(Nat.1) = Nat.2
                    Nat.2.pow(Nat.1) - Nat.1 = Nat.2 - Nat.1
                    Nat.2 - Nat.1 = Nat.1
                    stirling(m.suc, Nat.2) =
                        Nat.2.pow(m.suc - Nat.1) - Nat.1
                }
            }
            p(m.suc)
        }
    }
    alt_induction(p)
    p(Nat.0) and forall(m: Nat) { p(m) implies p(m.suc) }
    forall(m: Nat) { p(m) }
    if forall(m: Nat) { p(m) } {
        forall(m: Nat) {
            p(m) = (Nat.0 < m implies stirling(m, Nat.2) = Nat.2.pow(m - Nat.1) - Nat.1)
            Nat.0 < m implies stirling(m, Nat.2) = Nat.2.pow(m - Nat.1) - Nat.1
        }
        stirling_two_all(n)
        if Nat.0 < n {
            Nat.0 < n implies stirling(n, Nat.2) = Nat.2.pow(n - Nat.1) - Nat.1
            stirling(n, Nat.2) = Nat.2.pow(n - Nat.1) - Nat.1
        }
    }
    if not forall(m: Nat) { p(m) } {
        false
    }
}


/// A falling product with more factors than the base vanishes: for k >= n,
/// falling_product(n, k) = 0.
theorem falling_product_ge(n: Nat, k: Nat) {
    n <= k implies falling_product(n, k) = Nat.0
} by {
    define p(m: Nat) -> Bool {
        forall(n2: Nat) {
            n2 <= m implies falling_product(n2, m) = Nat.0
        }
    }
    // base m = 0
    forall(n2: Nat) {
        if n2 <= Nat.0 {
            only_zero_lte_zero(n2)
            n2 = Nat.0
            falling_product(n2, Nat.0) = n2
            falling_product(n2, Nat.0) = Nat.0
        }
    }
    p(Nat.0)
    // step
    forall(m: Nat) {
        if p(m) {
            forall(n2: Nat) {
                if n2 <= m.suc {
                    if n2 = m.suc {
                        falling_product_suc(n2, m)
                        falling_product(n2, m.suc) = falling_product(n2, m) * (n2 - m.suc)
                        n2 - m.suc = Nat.0
                        falling_product(n2, m.suc) = falling_product(n2, m) * Nat.0
                        falling_product(n2, m) * Nat.0 = Nat.0
                        falling_product(n2, m.suc) = Nat.0
                    } else {
                        n2 != m.suc
                        lt_or_lte(m, n2)
                        if m < n2 {
                            lt_imp_lte_suc(m, n2)
                            m.suc <= n2
                            lte_antisymm(n2, m.suc)
                            n2 <= m.suc and m.suc <= n2
                            n2 = m.suc
                            false
                        }
                        if not m < n2 {
                            n2 <= m
                            p(m)
                            if forall(n3: Nat) {
                                n3 <= m implies falling_product(n3, m) = Nat.0
                            } {
                                n2 <= m implies falling_product(n2, m) = Nat.0
                                falling_product(n2, m) = Nat.0
                                falling_product_suc(n2, m)
                                falling_product(n2, m.suc) = falling_product(n2, m) * (n2 - m.suc)
                                falling_product(n2, m.suc) = Nat.0 * (n2 - m.suc)
                                Nat.0 * (n2 - m.suc) = Nat.0
                                falling_product(n2, m.suc) = Nat.0
                            }
                            if not forall(n3: Nat) {
                                n3 <= m implies falling_product(n3, m) = Nat.0
                            } {
                                false
                            }
                            falling_product(n2, m.suc) = Nat.0
                        }
                        falling_product(n2, m.suc) = Nat.0
                    }
                }
            }
            p(m.suc)
        }
    }
    alt_induction(p)
    p(Nat.0) and forall(m: Nat) { p(m) implies p(m.suc) }
    forall(m: Nat) { p(m) }
    if forall(m: Nat) { p(m) } {
        if n <= k {
            p(k)
            forall(n2: Nat) {
                n2 <= k implies falling_product(n2, k) = Nat.0
            }
            falling_product(n, k) = Nat.0
        }
    }
    if not forall(m: Nat) { p(m) } {
        false
    }
}

/// n * falling_product(n, k) = falling_product(n, k + 1) + (k + 1) * falling_product(n, k).
theorem falling_mul_suc(n: Nat, k: Nat) {
    n * falling_product(n, k) = falling_product(n, k.suc) + k.suc * falling_product(n, k)
} by {
    if k.suc <= n {
        add_sub(n, k.suc)
        (n - k.suc) + k.suc = n
        distrib_right(n - k.suc, k.suc, falling_product(n, k))
        ((n - k.suc) + k.suc) * falling_product(n, k) =
            (n - k.suc) * falling_product(n, k) + k.suc * falling_product(n, k)
        n * falling_product(n, k) =
            (n - k.suc) * falling_product(n, k) + k.suc * falling_product(n, k)
        falling_product_suc(n, k)
        falling_product(n, k.suc) = falling_product(n, k) * (n - k.suc)
        mul_comm(falling_product(n, k), n - k.suc)
        falling_product(n, k) * (n - k.suc) = (n - k.suc) * falling_product(n, k)
        n * falling_product(n, k) = falling_product(n, k.suc) + k.suc * falling_product(n, k)
    } else {
        not k.suc <= n
        n < k.suc
        lt_imp_lte_suc(n, k)
        n <= k
        falling_product_ge(n, k)
        falling_product(n, k) = Nat.0
        n * falling_product(n, k) = n * Nat.0
        n * Nat.0 = Nat.0
        falling_product_ge(n, k.suc)
        n <= k.suc
        falling_product(n, k.suc) = Nat.0
        k.suc * falling_product(n, k) = k.suc * Nat.0
        k.suc * Nat.0 = Nat.0
        falling_product(n, k.suc) + k.suc * falling_product(n, k) = Nat.0 + Nat.0
        Nat.0 + Nat.0 = Nat.0
        n * falling_product(n, k) = falling_product(n, k.suc) + k.suc * falling_product(n, k)
    }
}

/// The k-th summand of the falling factorial expansion of n^m:
/// S(m, k + 1) * falling_product(n, k).
define stirling_falling_term(m: Nat, n: Nat, k: Nat) -> Nat {
    stirling(m, k.suc) * falling_product(n, k)
}

/// The shifted summand: S(m, k + 1) * falling_product(n, k + 1).
define stirling_falling_shift(m: Nat, n: Nat, k: Nat) -> Nat {
    stirling(m, k.suc) * falling_product(n, k.suc)
}

/// The weighted summand: (k + 1) * S(m, k + 1) * falling_product(n, k).
define stirling_falling_weight(m: Nat, n: Nat, k: Nat) -> Nat {
    k.suc * stirling_falling_term(m, n, k)
}

/// The lower-index summand: S(m, k) * falling_product(n, k).
define stirling_falling_lower(m: Nat, n: Nat, k: Nat) -> Nat {
    stirling(m, k) * falling_product(n, k)
}

/// n times the k-th summand splits into the shifted and weighted parts.
theorem stirling_falling_split(n: Nat, m: Nat, k: Nat) {
    n * stirling_falling_term(m, n, k) =
        stirling_falling_shift(m, n, k) + stirling_falling_weight(m, n, k)
} by {
    stirling_falling_term(m, n, k) = stirling(m, k.suc) * falling_product(n, k)
    n * stirling_falling_term(m, n, k) = n * (stirling(m, k.suc) * falling_product(n, k))
    mul_assoc(n, stirling(m, k.suc), falling_product(n, k))
    (n * stirling(m, k.suc)) * falling_product(n, k) =
        n * (stirling(m, k.suc) * falling_product(n, k))
    n * stirling_falling_term(m, n, k) =
        (n * stirling(m, k.suc)) * falling_product(n, k)
    mul_comm(n, stirling(m, k.suc))
    n * stirling(m, k.suc) = stirling(m, k.suc) * n
    n * stirling_falling_term(m, n, k) =
        (stirling(m, k.suc) * n) * falling_product(n, k)
    mul_assoc(stirling(m, k.suc), n, falling_product(n, k))
    (stirling(m, k.suc) * n) * falling_product(n, k) =
        stirling(m, k.suc) * (n * falling_product(n, k))
    n * stirling_falling_term(m, n, k) =
        stirling(m, k.suc) * (n * falling_product(n, k))
    falling_mul_suc(n, k)
    n * falling_product(n, k) =
        falling_product(n, k.suc) + k.suc * falling_product(n, k)
    stirling(m, k.suc) * (n * falling_product(n, k)) =
        stirling(m, k.suc) *
        (falling_product(n, k.suc) + k.suc * falling_product(n, k))
    distrib_left(stirling(m, k.suc), falling_product(n, k.suc),
        k.suc * falling_product(n, k))
    stirling(m, k.suc) *
        (falling_product(n, k.suc) + k.suc * falling_product(n, k)) =
        stirling(m, k.suc) * falling_product(n, k.suc) +
        stirling(m, k.suc) * (k.suc * falling_product(n, k))
    n * stirling_falling_term(m, n, k) =
        stirling(m, k.suc) * falling_product(n, k.suc) +
        stirling(m, k.suc) * (k.suc * falling_product(n, k))
    stirling_falling_shift(m, n, k) =
        stirling(m, k.suc) * falling_product(n, k.suc)
    stirling_falling_weight(m, n, k) = k.suc * stirling_falling_term(m, n, k)
    stirling_falling_term(m, n, k) = stirling(m, k.suc) * falling_product(n, k)
    stirling_falling_weight(m, n, k) = k.suc * (stirling(m, k.suc) * falling_product(n, k))
    mul_assoc(k.suc, stirling(m, k.suc), falling_product(n, k))
    (k.suc * stirling(m, k.suc)) * falling_product(n, k) =
        k.suc * (stirling(m, k.suc) * falling_product(n, k))
    stirling_falling_weight(m, n, k) =
        (k.suc * stirling(m, k.suc)) * falling_product(n, k)
    mul_comm(k.suc, stirling(m, k.suc))
    k.suc * stirling(m, k.suc) = stirling(m, k.suc) * k.suc
    stirling_falling_weight(m, n, k) =
        (stirling(m, k.suc) * k.suc) * falling_product(n, k)
    mul_assoc(stirling(m, k.suc), k.suc, falling_product(n, k))
    (stirling(m, k.suc) * k.suc) * falling_product(n, k) =
        stirling(m, k.suc) * (k.suc * falling_product(n, k))
    stirling_falling_weight(m, n, k) =
        stirling(m, k.suc) * (k.suc * falling_product(n, k))
    n * stirling_falling_term(m, n, k) =
        stirling_falling_shift(m, n, k) + stirling_falling_weight(m, n, k)
}

/// The shifted sum reindexes to the lower-index sum: for m > 0,
/// sum_{k=0}^{m-1} S(m, k+1) * fp(n, k+1) = sum_{k=0}^{m} S(m, k) * fp(n, k).
theorem stirling_falling_shift_sum(m: Nat, n: Nat) {
    Nat.0 < m implies
        partial(stirling_falling_shift(m, n), m) =
            partial(stirling_falling_lower(m, n), m.suc)
} by {
    if Nat.0 < m {
        // pointwise: shift(k) = lower(k + 1)
        forall(k: Nat) {
            if k < m {
                stirling_falling_shift(m, n, k) =
                    stirling(m, k.suc) * falling_product(n, k.suc)
                stirling_falling_lower(m, n, k.suc) =
                    stirling(m, k.suc) * falling_product(n, k.suc)
                stirling_falling_shift(m, n, k) =
                    stirling_falling_lower(m, n, k.suc)
                compose(stirling_falling_lower(m, n), Nat.suc, k) =
                    stirling_falling_lower(m, n, k.suc)
                stirling_falling_shift(m, n, k) =
                    compose(stirling_falling_lower(m, n), Nat.suc, k)
            }
        }
        partial_pointwise_eq[Nat](stirling_falling_shift(m, n),
            compose(stirling_falling_lower(m, n), Nat.suc), m)
        partial(stirling_falling_shift(m, n), m) =
            partial(compose(stirling_falling_lower(m, n), Nat.suc), m)
        // shift_suc: lower(0) + partial(compose(lower, suc), m) = partial(lower, m.suc)
        partial_shift_suc[Nat](stirling_falling_lower(m, n), m)
        stirling_falling_lower(m, n, Nat.0) +
            partial(compose(stirling_falling_lower(m, n), Nat.suc), m) =
            partial(stirling_falling_lower(m, n), m.suc)
        stirling_falling_lower(m, n, Nat.0) =
            stirling(m, Nat.0) * falling_product(n, Nat.0)
        stirling_zero_col(m)
        stirling(m, Nat.0) = Nat.0
        stirling_falling_lower(m, n, Nat.0) = Nat.0 * falling_product(n, Nat.0)
        Nat.0 * falling_product(n, Nat.0) = Nat.0
        stirling_falling_lower(m, n, Nat.0) = Nat.0
        Nat.0 + partial(compose(stirling_falling_lower(m, n), Nat.suc), m) =
            partial(stirling_falling_lower(m, n), m.suc)
        Nat.0 + partial(compose(stirling_falling_lower(m, n), Nat.suc), m) =
            partial(compose(stirling_falling_lower(m, n), Nat.suc), m)
        partial(compose(stirling_falling_lower(m, n), Nat.suc), m) =
            partial(stirling_falling_lower(m, n), m.suc)
        partial(stirling_falling_shift(m, n), m) =
            partial(stirling_falling_lower(m, n), m.suc)
    }
}

/// The top weighted summand vanishes: for m > 0,
/// partial(weight, m.suc) = partial(weight, m).
theorem stirling_falling_weight_top(m: Nat, n: Nat) {
    Nat.0 < m implies
        partial(stirling_falling_weight(m, n), m.suc) =
            partial(stirling_falling_weight(m, n), m)
} by {
    if Nat.0 < m {
        partial_split_last(stirling_falling_weight(m, n), m)
        partial(stirling_falling_weight(m, n), m.suc) =
            partial(stirling_falling_weight(m, n), m) +
            stirling_falling_weight(m, n, m)
        stirling_falling_weight(m, n, m) =
            m.suc * stirling_falling_term(m, n, m)
        stirling_falling_term(m, n, m) =
            stirling(m, m.suc) * falling_product(n, m)
        m < m.suc
        stirling_out_of_bounds(m, m.suc)
        stirling(m, m.suc) = Nat.0
        stirling_falling_term(m, n, m) = Nat.0 * falling_product(n, m)
        Nat.0 * falling_product(n, m) = Nat.0
        stirling_falling_term(m, n, m) = Nat.0
        stirling_falling_weight(m, n, m) = m.suc * Nat.0
        m.suc * Nat.0 = Nat.0
        stirling_falling_weight(m, n, m) = Nat.0
        partial(stirling_falling_weight(m, n), m.suc) =
            partial(stirling_falling_weight(m, n), m) + Nat.0
        partial(stirling_falling_weight(m, n), m) + Nat.0 =
            partial(stirling_falling_weight(m, n), m)
        partial(stirling_falling_weight(m, n), m.suc) =
            partial(stirling_falling_weight(m, n), m)
    }
}

/// The k-th summand of the (m+1)-row falls into the lower and weighted parts.
theorem stirling_falling_row_suc(m: Nat, n: Nat, k: Nat) {
    stirling_falling_term(m.suc, n, k) =
        stirling_falling_lower(m, n, k) + stirling_falling_weight(m, n, k)
} by {
    stirling_falling_term(m.suc, n, k) =
        stirling(m.suc, k.suc) * falling_product(n, k)
    stirling_recurrence(m, k)
    stirling(m.suc, k.suc) = stirling(m, k) + k.suc * stirling(m, k.suc)
    stirling_falling_term(m.suc, n, k) =
        (stirling(m, k) + k.suc * stirling(m, k.suc)) * falling_product(n, k)
    distrib_right(stirling(m, k), k.suc * stirling(m, k.suc), falling_product(n, k))
    (stirling(m, k) + k.suc * stirling(m, k.suc)) * falling_product(n, k) =
        stirling(m, k) * falling_product(n, k) +
        (k.suc * stirling(m, k.suc)) * falling_product(n, k)
    stirling_falling_term(m.suc, n, k) =
        stirling(m, k) * falling_product(n, k) +
        (k.suc * stirling(m, k.suc)) * falling_product(n, k)
    stirling_falling_lower(m, n, k) = stirling(m, k) * falling_product(n, k)
    stirling_falling_weight(m, n, k) = k.suc * stirling_falling_term(m, n, k)
    stirling_falling_term(m, n, k) = stirling(m, k.suc) * falling_product(n, k)
    stirling_falling_weight(m, n, k) = k.suc * (stirling(m, k.suc) * falling_product(n, k))
    mul_assoc(k.suc, stirling(m, k.suc), falling_product(n, k))
    (k.suc * stirling(m, k.suc)) * falling_product(n, k) =
        k.suc * (stirling(m, k.suc) * falling_product(n, k))
    stirling_falling_weight(m, n, k) =
        (k.suc * stirling(m, k.suc)) * falling_product(n, k)
    stirling_falling_term(m.suc, n, k) =
        stirling_falling_lower(m, n, k) + stirling_falling_weight(m, n, k)
}

/// The classical identity: n^m = sum_{k=0}^{m-1} S(m, k+1) * falling_product(n, k)
/// for positive m.
theorem stirling_falling_pow(n: Nat, m: Nat) {
    Nat.0 < m implies n.pow(m) = partial(stirling_falling_term(m, n), m)
} by {
    define p(x: Nat) -> Bool {
        Nat.0 < x implies n.pow(x) = partial(stirling_falling_term(x, n), x)
    }
    // base m = 1
    pow_one[Nat](n)
    n.pow(Nat.1) = n
    partial(stirling_falling_term(Nat.1, n), Nat.1) =
        stirling_falling_term(Nat.1, n, Nat.0)
    stirling_falling_term(Nat.1, n, Nat.0) =
        stirling(Nat.1, Nat.1) * falling_product(n, Nat.0)
    stirling_one_one
    stirling(Nat.1, Nat.1) = Nat.1
    falling_product_zero(n)
    falling_product(n, Nat.0) = n
    stirling_falling_term(Nat.1, n, Nat.0) = Nat.1 * n
    Nat.1 * n = n
    stirling_falling_term(Nat.1, n, Nat.0) = n
    partial(stirling_falling_term(Nat.1, n), Nat.1) = n
    n.pow(Nat.1) = partial(stirling_falling_term(Nat.1, n), Nat.1)
    p(Nat.1)
    // p(0) is vacuous
    if Nat.0 < Nat.0 {
        not_lt_zero(Nat.0)
        not (Nat.0 < Nat.0)
        false
    }
    p(Nat.0)
    // step
    forall(t: Nat) {
        if p(t) {
            if Nat.0 < t.suc {
                if Nat.0 < t {
                    p(t)
                    p(t) = (Nat.0 < t implies n.pow(t) = partial(stirling_falling_term(t, n), t))
                    n.pow(t) = partial(stirling_falling_term(t, n), t)
                    pow_add(n, t, Nat.1)
                    n.pow(t) * n.pow(Nat.1) = n.pow(t + Nat.1)
                    pow_one[Nat](n)
                    n.pow(Nat.1) = n
                    n.pow(t) * n = n.pow(t.suc)
                    mul_comm(n.pow(t), n)
                    n.pow(t) * n = n * n.pow(t)
                    n.pow(t.suc) = n * n.pow(t)
                    n.pow(t.suc) = n * partial(stirling_falling_term(t, n), t)
                    partial_scalar_mul[Nat](n, stirling_falling_term(t, n), t)
                    n * partial(stirling_falling_term(t, n), t) =
                        partial(mul_fn(n, stirling_falling_term(t, n)), t)
                    n.pow(t.suc) = partial(mul_fn(n, stirling_falling_term(t, n)), t)
                    // pointwise: n * term(k) = shift(k) + weight(k)
                    forall(k: Nat) {
                        if k < t {
                            stirling_falling_split(n, t, k)
                            n * stirling_falling_term(t, n, k) =
                                stirling_falling_shift(t, n, k) +
                                stirling_falling_weight(t, n, k)
                            mul_fn(n, stirling_falling_term(t, n), k) =
                                n * stirling_falling_term(t, n, k)
                            mul_fn(n, stirling_falling_term(t, n), k) =
                                stirling_falling_shift(t, n, k) +
                                stirling_falling_weight(t, n, k)
                            add_fn(stirling_falling_shift(t, n),
                                stirling_falling_weight(t, n), k) =
                                stirling_falling_shift(t, n, k) +
                                stirling_falling_weight(t, n, k)
                            mul_fn(n, stirling_falling_term(t, n), k) =
                                add_fn(stirling_falling_shift(t, n),
                                    stirling_falling_weight(t, n), k)
                        }
                    }
                    partial_pointwise_eq[Nat](mul_fn(n, stirling_falling_term(t, n)),
                        add_fn(stirling_falling_shift(t, n), stirling_falling_weight(t, n)), t)
                    partial(mul_fn(n, stirling_falling_term(t, n)), t) =
                        partial(add_fn(stirling_falling_shift(t, n),
                            stirling_falling_weight(t, n)), t)
                    partial_add(stirling_falling_shift(t, n),
                        stirling_falling_weight(t, n), t)
                    partial(stirling_falling_shift(t, n), t) +
                        partial(stirling_falling_weight(t, n), t) =
                        partial(add_fn(stirling_falling_shift(t, n),
                            stirling_falling_weight(t, n)), t)
                    n.pow(t.suc) =
                        partial(stirling_falling_shift(t, n), t) +
                        partial(stirling_falling_weight(t, n), t)
                    stirling_falling_shift_sum(t, n)
                    partial(stirling_falling_shift(t, n), t) =
                        partial(stirling_falling_lower(t, n), t.suc)
                    stirling_falling_weight_top(t, n)
                    partial(stirling_falling_weight(t, n), t.suc) =
                        partial(stirling_falling_weight(t, n), t)
                    n.pow(t.suc) =
                        partial(stirling_falling_lower(t, n), t.suc) +
                        partial(stirling_falling_weight(t, n), t.suc)
                    // target: partial(term(t.suc, n), t.suc.suc)
                    forall(k: Nat) {
                        if k < t.suc.suc {
                            stirling_falling_row_suc(t, n, k)
                            stirling_falling_term(t.suc, n, k) =
                                stirling_falling_lower(t, n, k) +
                                stirling_falling_weight(t, n, k)
                            add_fn(stirling_falling_lower(t, n),
                                stirling_falling_weight(t, n), k) =
                                stirling_falling_lower(t, n, k) +
                                stirling_falling_weight(t, n, k)
                            stirling_falling_term(t.suc, n, k) =
                                add_fn(stirling_falling_lower(t, n),
                                    stirling_falling_weight(t, n), k)
                        }
                    }
                    partial_pointwise_eq[Nat](stirling_falling_term(t.suc, n),
                        add_fn(stirling_falling_lower(t, n), stirling_falling_weight(t, n)),
                        t.suc.suc)
                    partial(stirling_falling_term(t.suc, n), t.suc.suc) =
                        partial(add_fn(stirling_falling_lower(t, n),
                            stirling_falling_weight(t, n)), t.suc.suc)
                    partial_add(stirling_falling_lower(t, n),
                        stirling_falling_weight(t, n), t.suc.suc)
                    partial(stirling_falling_lower(t, n), t.suc.suc) +
                        partial(stirling_falling_weight(t, n), t.suc.suc) =
                        partial(add_fn(stirling_falling_lower(t, n),
                            stirling_falling_weight(t, n)), t.suc.suc)
                    partial(stirling_falling_lower(t, n), t.suc.suc) +
                        partial(stirling_falling_weight(t, n), t.suc.suc) =
                        partial(stirling_falling_term(t.suc, n), t.suc.suc)
                    // the top summands of the target sums vanish
                    partial_split_last(stirling_falling_lower(t, n), t.suc)
                    partial(stirling_falling_lower(t, n), t.suc.suc) =
                        partial(stirling_falling_lower(t, n), t.suc) +
                        stirling_falling_lower(t, n, t.suc)
                    stirling_falling_lower(t, n, t.suc) =
                        stirling(t, t.suc) * falling_product(n, t.suc)
                    t < t.suc
                    stirling_out_of_bounds(t, t.suc)
                    stirling(t, t.suc) = Nat.0
                    stirling_falling_lower(t, n, t.suc) =
                        Nat.0 * falling_product(n, t.suc)
                    Nat.0 * falling_product(n, t.suc) = Nat.0
                    stirling_falling_lower(t, n, t.suc) = Nat.0
                    partial(stirling_falling_lower(t, n), t.suc.suc) =
                        partial(stirling_falling_lower(t, n), t.suc) + Nat.0
                    partial(stirling_falling_lower(t, n), t.suc) + Nat.0 =
                        partial(stirling_falling_lower(t, n), t.suc)
                    partial(stirling_falling_lower(t, n), t.suc.suc) =
                        partial(stirling_falling_lower(t, n), t.suc)
                    partial_split_last(stirling_falling_weight(t, n), t.suc)
                    partial(stirling_falling_weight(t, n), t.suc.suc) =
                        partial(stirling_falling_weight(t, n), t.suc) +
                        stirling_falling_weight(t, n, t.suc)
                    stirling_falling_weight(t, n, t.suc) =
                        t.suc.suc * stirling_falling_term(t, n, t.suc)
                    stirling_falling_term(t, n, t.suc) =
                        stirling(t, t.suc.suc) * falling_product(n, t.suc)
                    t < t.suc.suc
                    stirling_out_of_bounds(t, t.suc.suc)
                    stirling(t, t.suc.suc) = Nat.0
                    stirling_falling_term(t, n, t.suc) =
                        Nat.0 * falling_product(n, t.suc)
                    Nat.0 * falling_product(n, t.suc) = Nat.0
                    stirling_falling_term(t, n, t.suc) = Nat.0
                    stirling_falling_weight(t, n, t.suc) = t.suc.suc * Nat.0
                    t.suc.suc * Nat.0 = Nat.0
                    stirling_falling_weight(t, n, t.suc) = Nat.0
                    partial(stirling_falling_weight(t, n), t.suc.suc) =
                        partial(stirling_falling_weight(t, n), t.suc) + Nat.0
                    partial(stirling_falling_weight(t, n), t.suc) + Nat.0 =
                        partial(stirling_falling_weight(t, n), t.suc)
                    partial(stirling_falling_weight(t, n), t.suc.suc) =
                        partial(stirling_falling_weight(t, n), t.suc)
                    partial(stirling_falling_lower(t, n), t.suc.suc) +
                        partial(stirling_falling_weight(t, n), t.suc.suc) =
                        partial(stirling_falling_lower(t, n), t.suc) +
                        partial(stirling_falling_weight(t, n), t.suc)
                    n.pow(t.suc) =
                        partial(stirling_falling_lower(t, n), t.suc) +
                        partial(stirling_falling_weight(t, n), t.suc)
                    partial_split_last(stirling_falling_term(t.suc, n), t.suc)
                    partial(stirling_falling_term(t.suc, n), t.suc.suc) =
                        partial(stirling_falling_term(t.suc, n), t.suc) +
                        stirling_falling_term(t.suc, n, t.suc)
                    stirling_falling_term(t.suc, n, t.suc) =
                        stirling(t.suc, t.suc.suc) * falling_product(n, t.suc)
                    t.suc < t.suc.suc
                    stirling_out_of_bounds(t.suc, t.suc.suc)
                    stirling(t.suc, t.suc.suc) = Nat.0
                    stirling_falling_term(t.suc, n, t.suc) =
                        Nat.0 * falling_product(n, t.suc)
                    Nat.0 * falling_product(n, t.suc) = Nat.0
                    stirling_falling_term(t.suc, n, t.suc) = Nat.0
                    partial(stirling_falling_term(t.suc, n), t.suc.suc) =
                        partial(stirling_falling_term(t.suc, n), t.suc) + Nat.0
                    partial(stirling_falling_term(t.suc, n), t.suc) + Nat.0 =
                        partial(stirling_falling_term(t.suc, n), t.suc)
                    partial(stirling_falling_term(t.suc, n), t.suc.suc) =
                        partial(stirling_falling_term(t.suc, n), t.suc)
                    n.pow(t.suc) =
                        partial(stirling_falling_term(t.suc, n), t.suc)
                } else {
                    Nat.0 = t
                    pow_one[Nat](n)
                    n.pow(Nat.1) = n
                    partial(stirling_falling_term(Nat.1, n), Nat.1) = n
                    n.pow(t.suc) = n
                    t.suc = Nat.1
                    t.suc.suc = Nat.2
                    partial_split_last(stirling_falling_term(Nat.1, n), Nat.1)
                    partial(stirling_falling_term(Nat.1, n), Nat.2) =
                        partial(stirling_falling_term(Nat.1, n), Nat.1) +
                        stirling_falling_term(Nat.1, n, Nat.1)
                    stirling_falling_term(Nat.1, n, Nat.1) =
                        stirling(Nat.1, Nat.2) * falling_product(n, Nat.1)
                    stirling_out_of_bounds(Nat.1, Nat.2)
                    Nat.1 < Nat.2
                    stirling(Nat.1, Nat.2) = Nat.0
                    stirling_falling_term(Nat.1, n, Nat.1) =
                        Nat.0 * falling_product(n, Nat.1)
                    Nat.0 * falling_product(n, Nat.1) = Nat.0
                    stirling_falling_term(Nat.1, n, Nat.1) = Nat.0
                    partial(stirling_falling_term(Nat.1, n), Nat.2) = n + Nat.0
                    n + Nat.0 = n
                    partial(stirling_falling_term(Nat.1, n), Nat.2) = n
                    partial_split_last(stirling_falling_term(Nat.1, n), Nat.1)
                    partial(stirling_falling_term(Nat.1, n), Nat.2) =
                        partial(stirling_falling_term(Nat.1, n), Nat.1) +
                        stirling_falling_term(Nat.1, n, Nat.1)
                    stirling_falling_term(Nat.1, n, Nat.1) =
                        stirling(Nat.1, Nat.2) * falling_product(n, Nat.1)
                    stirling_out_of_bounds(Nat.1, Nat.2)
                    Nat.1 < Nat.2
                    stirling(Nat.1, Nat.2) = Nat.0
                    stirling_falling_term(Nat.1, n, Nat.1) =
                        Nat.0 * falling_product(n, Nat.1)
                    Nat.0 * falling_product(n, Nat.1) = Nat.0
                    stirling_falling_term(Nat.1, n, Nat.1) = Nat.0
                    partial(stirling_falling_term(Nat.1, n), Nat.2) = n + Nat.0
                    n + Nat.0 = n
                    partial(stirling_falling_term(Nat.1, n), Nat.2) = n
                    n.pow(t.suc) = partial(stirling_falling_term(t.suc, n), t.suc)
                }
            }
            Nat.0 < t.suc implies n.pow(t.suc) = partial(stirling_falling_term(t.suc, n), t.suc)
            p(t.suc) = (Nat.0 < t.suc implies n.pow(t.suc) = partial(stirling_falling_term(t.suc, n), t.suc))
            p(t.suc)
        }
    }
    alt_induction(p)
    p(Nat.0) and forall(t: Nat) { p(t) implies p(t.suc) }
    forall(t: Nat) { p(t) }
    if forall(t: Nat) { p(t) } {
        if Nat.0 < m {
            p(m)
            p(m) = (Nat.0 < m implies n.pow(m) = partial(stirling_falling_term(m, n), m))
            n.pow(m) = partial(stirling_falling_term(m, n), m)
        }
    }
    if not forall(t: Nat) { p(t) } {
        false
    }
}
