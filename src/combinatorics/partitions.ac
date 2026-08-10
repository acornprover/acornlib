/// Integer partitions.
///
/// A partition of a natural number `n` is a multiset of positive integers
/// summing to `n`.  A multiset is represented by the list of its parts in
/// non-increasing order, so that every partition has exactly one
/// representation as a list.  The number of partitions of `n` is written
/// `p(n)`.
///
/// This file proves the small values
/// p(0) = 1, p(1) = 1, p(2) = 2, p(3) = 3 and p(4) = 5,
/// verified by explicit enumeration of the partitions (the sets of
/// partitions of 0..4 are characterized exactly), and records the classic
/// identities — the generating function identity and Euler's pentagonal
/// number theorem — as statements for later work.

from nat import Nat, add_comm, add_assoc, add_cancels_left, add_cancels_right,
    add_sub, sub_self, lte_trans, lte_suc_suc, lt_suc, lt_imp_lte_suc,
    lt_imp_lt_suc, lt_or_lte, lte_antisymm, lte_cancel_suc, lt_cancel_suc,
    add_to_zero, lte_add_left, lte_add_right, lt_add_left, lte_ref,
    lte_and_lt, lt_and_lte, lt_not_symm, lte_imp_not_lt, lt_trans, one_plus_one,
    add_one_right, add_one_left, add_zero_left, lt_not_ref, not_lt_zero,
    lt_suc_right, trichotomy
from list import List, sum, map, add_contains_left, add_contains_right,
    filter_contains_and, filter_contained_by_and, filter_equivalent_to_and,
    unique_implies_tail_unique, unique_length, singleton_unique,
    map_contains_of_contains, range_contains_of_lt
from data.basic.logic import or_intro_left, or_intro_right
from data.list.list_cons_membership import cons_contains_eq, cons_contains_of_tail_contains, cons_contains_head, nil_not_contains
from list import cons_unique_of_tail_unique_not_contains
from data.basic.set import Set, contains_set_intro, contains_set_at, finite_constraint,
    cardinality_always_exists, cardinality_is_well_defined

numerals Nat

/// True if every element of the list is at most `head`, and the tail is
/// non-increasing.
define non_increasing_from(head: Nat, tail: List[Nat]) -> Bool {
    match tail {
        List.nil {
            true
        }
        List.cons(head2, tail2) {
            head2 <= head and non_increasing_from(head2, tail2)
        }
    }
}

/// True if the elements of the list are in non-increasing order.
define non_increasing(parts: List[Nat]) -> Bool {
    match parts {
        List.nil {
            true
        }
        List.cons(head, tail) {
            non_increasing_from(head, tail)
        }
    }
}

/// True if every element of the list is positive.
define all_positive(parts: List[Nat]) -> Bool {
    match parts {
        List.nil {
            true
        }
        List.cons(head, tail) {
            Nat.0 < head and all_positive(tail)
        }
    }
}

/// True if `parts` is a partition of `n`: a list of positive integers, in
/// non-increasing order, whose sum is `n`.
define is_partition(parts: List[Nat], n: Nat) -> Bool {
    sum(parts) = n and non_increasing(parts) and all_positive(parts)
}

/// Zero is at most every natural number.
theorem zero_lte(a: Nat) {
    Nat.0 <= a
} by {
    add_zero_left(a)
    Nat.0 + a = a
    exists(c: Nat) {
        Nat.0 + c = a
    }
}

/// A partition of `n` sums to `n`.
theorem is_partition_sum(parts: List[Nat], n: Nat) {
    is_partition(parts, n) implies sum(parts) = n
} by {
    if is_partition(parts, n) {
        sum(parts) = n and non_increasing(parts) and all_positive(parts)
        sum(parts) = n
    }
}

/// A partition of `n` is listed in non-increasing order.
theorem is_partition_non_increasing(parts: List[Nat], n: Nat) {
    is_partition(parts, n) implies non_increasing(parts)
} by {
    if is_partition(parts, n) {
        non_increasing(parts)
    }
}

/// Every part of a partition of `n` is positive.
theorem is_partition_all_positive(parts: List[Nat], n: Nat) {
    is_partition(parts, n) implies all_positive(parts)
} by {
    if is_partition(parts, n) {
        all_positive(parts)
    }
}

/// A positive non-increasing list summing to `n` is a partition of `n`.
theorem is_partition_intro(parts: List[Nat], n: Nat) {
    sum(parts) = n and non_increasing(parts) and all_positive(parts)
    implies is_partition(parts, n)
} by {
    if sum(parts) = n and non_increasing(parts) and all_positive(parts) {
        is_partition(parts, n)
    }
}

/// The head of a positive list is positive.
theorem all_positive_head(a: Nat, t: List[Nat]) {
    all_positive(List.cons(a, t)) implies Nat.0 < a
} by {
    if all_positive(List.cons(a, t)) {
        Nat.0 < a
    }
}

/// The tail of a positive list is positive.
theorem all_positive_tail(a: Nat, t: List[Nat]) {
    all_positive(List.cons(a, t)) implies all_positive(t)
} by {
    if all_positive(List.cons(a, t)) {
        all_positive(t)
    }
}

/// The head constraint of a non-increasing tail: the second element is at
/// most the first.
theorem non_increasing_from_second(head: Nat, head2: Nat, tail: List[Nat]) {
    non_increasing_from(head, List.cons(head2, tail)) implies head2 <= head
}

/// The tail of a non-increasing tail is itself non-increasing.
theorem non_increasing_from_tail(head: Nat, head2: Nat, tail: List[Nat]) {
    non_increasing_from(head, List.cons(head2, tail)) implies non_increasing_from(head2, tail)
}

/// The second element of a non-increasing list is at most the first.
theorem non_increasing_second(head: Nat, head2: Nat, tail: List[Nat]) {
    non_increasing(List.cons(head, List.cons(head2, tail))) implies head2 <= head
} by {
    non_increasing(List.cons(head, List.cons(head2, tail))) =
        non_increasing_from(head, List.cons(head2, tail))
    if non_increasing(List.cons(head, List.cons(head2, tail))) {
        non_increasing_from(head, List.cons(head2, tail))
        non_increasing_from_second(head, head2, tail)
        non_increasing_from(head, List.cons(head2, tail)) implies head2 <= head
        head2 <= head
    }
}

/// The tail of a non-increasing list is non-increasing.
theorem non_increasing_tail(head: Nat, head2: Nat, tail: List[Nat]) {
    non_increasing(List.cons(head, List.cons(head2, tail))) implies non_increasing(List.cons(head2, tail))

} by {
    non_increasing(List.cons(head, List.cons(head2, tail))) =
        non_increasing_from(head, List.cons(head2, tail))
    if non_increasing(List.cons(head, List.cons(head2, tail))) {
        non_increasing_from(head, List.cons(head2, tail))
        non_increasing_from_tail(head, head2, tail)
        non_increasing_from(head, List.cons(head2, tail)) implies non_increasing_from(head2, tail)
        non_increasing_from(head2, tail)
        non_increasing(List.cons(head2, tail)) = non_increasing_from(head2, tail)
        non_increasing(List.cons(head2, tail))
    }
}

/// A singleton list is non-increasing.
theorem ni_single(a: Nat) {
    non_increasing(List.cons(a, List.nil[Nat])) = true
} by {
    non_increasing(List.cons(a, List.nil[Nat])) = non_increasing_from(a, List.nil[Nat])
    non_increasing_from(a, List.nil[Nat]) = true
    non_increasing(List.cons(a, List.nil[Nat])) = true
}
/// A singleton list of a positive element is all-positive.
theorem ap_single(a: Nat) {
    Nat.0 < a implies all_positive(List.cons(a, List.nil[Nat])) = true
} by {
    if Nat.0 < a {
        all_positive(List.cons(a, List.nil[Nat])) = (Nat.0 < a and all_positive(List.nil[Nat]))
        all_positive(List.nil[Nat]) = true
        all_positive(List.cons(a, List.nil[Nat])) = true
    }
}
/// Two propositions equivalent in both directions are equal as booleans.
theorem bool_eq_of_iff(a: Bool, b: Bool) {
    (a implies b) and (b implies a) implies a = b
} by {
    if (a implies b) and (b implies a) {
        if a {
            if b {
                a = b
            }
            if not b {
                a implies b
                b
                false
            }
            a = b
        }
        if not a {
            if b {
                b implies a
                a
                false
            }
            if not b {
                a = b
            }
            a = b
        }
        a = b
    }
}

// ---------------------------------------------------------------------------
// Small values, by explicit enumeration.
// ---------------------------------------------------------------------------

/// The partitions of zero are exactly the empty list.
theorem partition_zero_members(l: List[Nat]) {
    is_partition(l, Nat.0) implies l = List.nil[Nat]
} by {
    if is_partition(l, Nat.0) {
        is_partition_sum(l, Nat.0)
        sum(l) = Nat.0
        match l {
            List.nil {
                l = List.nil[Nat]
            }
            List.cons(a, t) {
                sum(List.cons(a, t)) = a + sum(t)
                a + sum(t) = Nat.0
                add_to_zero(a, sum(t))
                a = Nat.0 and sum(t) = Nat.0
                a != Nat.0
                false
            }
        }
        l = List.nil[Nat]
    }
}

/// The empty list is the only partition of zero.
theorem partition_zero_sound(l: List[Nat]) {
    l = List.nil[Nat] implies is_partition(l, Nat.0)
} by {
    if l = List.nil[Nat] {
        sum(List.nil[Nat]) = Nat.0
        non_increasing(List.nil[Nat]) = true
        all_positive(List.nil[Nat]) = true
        is_partition_intro(List.nil[Nat], Nat.0)
        is_partition(List.nil[Nat], Nat.0)
        is_partition(l, Nat.0)
    }
}

/// The partitions of zero are exactly the empty list.
theorem partition_zero_char(l: List[Nat]) {
    is_partition(l, Nat.0) = (l = List.nil[Nat])
} by {
    partition_zero_members(l)
    is_partition(l, Nat.0) implies l = List.nil[Nat]
    partition_zero_sound(l)
    l = List.nil[Nat] implies is_partition(l, Nat.0)
    (is_partition(l, Nat.0) implies l = List.nil[Nat]) and
        (l = List.nil[Nat] implies is_partition(l, Nat.0))
    bool_eq_of_iff(is_partition(l, Nat.0), l = List.nil[Nat])
    is_partition(l, Nat.0) = (l = List.nil[Nat])
}

/// The partitions of one are exactly the singleton 1.
theorem partition_one_members(l: List[Nat]) {
    is_partition(l, Nat.1) implies l = List.cons(Nat.1, List.nil[Nat])
} by {
    if is_partition(l, Nat.1) {
        is_partition_sum(l, Nat.1)
        sum(l) = Nat.1
        match l {
            List.nil {
                sum(List.nil[Nat]) = Nat.0
                Nat.0 = Nat.1
                false
            }
            List.cons(a, t) {
                match t {
                    List.nil {
                        sum(List.cons(a, List.nil[Nat])) = a + sum(List.nil[Nat])
                        sum(List.nil[Nat]) = Nat.0
                        a + Nat.0 = a
                        sum(List.cons(a, List.nil[Nat])) = a
                        a = Nat.1
                        l = List.cons(a, List.nil[Nat])
                        l = List.cons(Nat.1, List.nil[Nat])
                    }
                    List.cons(b, t2) {
                        sum(List.cons(a, List.cons(b, t2))) = a + sum(List.cons(b, t2))
                        sum(List.cons(b, t2)) = b + sum(t2)
                        a + (b + sum(t2)) = Nat.1
                        is_partition_all_positive(l, Nat.1)
                        all_positive(List.cons(a, List.cons(b, t2)))
                        all_positive_head(a, List.cons(b, t2))
                        Nat.0 < a
                        lt_imp_lte_suc(Nat.0, a)
                        Nat.1 <= a
                        lte_add_right(Nat.1, Nat.1, a)
                        Nat.1 <= a implies Nat.1 + Nat.1 <= a + Nat.1
                        Nat.1 + Nat.1 <= a + Nat.1
                        Nat.1 + Nat.1 = Nat.2
                        Nat.2 <= a + Nat.1
                        add_one_right(a)
                        a + Nat.1 = a.suc
                        all_positive_tail(a, List.cons(b, t2))
                        all_positive(List.cons(b, t2))
                        all_positive_head(b, t2)
                        Nat.0 < b
                        lt_imp_lte_suc(Nat.0, b)
                        Nat.1 <= b
                        zero_lte(sum(t2))
                        Nat.0 <= sum(t2)
                        lte_add_left(b, Nat.0, sum(t2))
                        Nat.0 <= sum(t2) implies b + Nat.0 <= b + sum(t2)
                        b + Nat.0 <= b + sum(t2)
                        b + Nat.0 = b
                        b <= b + sum(t2)
                        lte_trans(Nat.1, b, b + sum(t2))
                        Nat.1 <= b + sum(t2)
                        lte_add_left(a, Nat.1, b + sum(t2))
                        Nat.1 <= b + sum(t2) implies a + Nat.1 <= a + (b + sum(t2))
                        a + Nat.1 <= a + (b + sum(t2))
                        a + (b + sum(t2)) = Nat.1
                        a + Nat.1 <= Nat.1
                        lte_trans(Nat.2, a + Nat.1, Nat.1)
                        Nat.2 <= Nat.1
                        lte_imp_not_lt(Nat.2, Nat.1)
                        Nat.2 <= Nat.1 implies not Nat.1 < Nat.2
                        not Nat.1 < Nat.2
                        lt_suc(Nat.1)
                        Nat.1 < Nat.2
                        false
                    }
                }
            }
        }
        l = List.cons(Nat.1, List.nil[Nat])
    }
}

/// The singleton 1 is the only partition of one.
theorem partition_one_sound(l: List[Nat]) {
    l = List.cons(Nat.1, List.nil[Nat]) implies is_partition(l, Nat.1)
} by {
    if l = List.cons(Nat.1, List.nil[Nat]) {
        sum(List.cons(Nat.1, List.nil[Nat])) = Nat.1 + sum(List.nil[Nat])
        sum(List.nil[Nat]) = Nat.0
        Nat.1 + Nat.0 = Nat.1
        sum(List.cons(Nat.1, List.nil[Nat])) = Nat.1
        non_increasing(List.cons(Nat.1, List.nil[Nat])) =
            non_increasing_from(Nat.1, List.nil[Nat])
        non_increasing_from(Nat.1, List.nil[Nat]) = true
        non_increasing(List.cons(Nat.1, List.nil[Nat])) = true
        all_positive(List.cons(Nat.1, List.nil[Nat])) = true
        is_partition_intro(List.cons(Nat.1, List.nil[Nat]), Nat.1)
        is_partition(List.cons(Nat.1, List.nil[Nat]), Nat.1)
        is_partition(l, Nat.1)
    }
}

/// The partitions of one are exactly the singleton 1.
theorem partition_one_char(l: List[Nat]) {
    is_partition(l, Nat.1) = (l = List.cons(Nat.1, List.nil[Nat]))
} by {
    partition_one_members(l)
    is_partition(l, Nat.1) implies l = List.cons(Nat.1, List.nil[Nat])
    partition_one_sound(l)
    l = List.cons(Nat.1, List.nil[Nat]) implies is_partition(l, Nat.1)
    (is_partition(l, Nat.1) implies l = List.cons(Nat.1, List.nil[Nat])) and
        (l = List.cons(Nat.1, List.nil[Nat]) implies is_partition(l, Nat.1))
    bool_eq_of_iff(is_partition(l, Nat.1), l = List.cons(Nat.1, List.nil[Nat]))
    is_partition(l, Nat.1) = (l = List.cons(Nat.1, List.nil[Nat]))
}

// ---------------------------------------------------------------------------
// Arithmetic helpers for the small-value case analysis.
// ---------------------------------------------------------------------------

/// A positive number below two is one.
theorem lt_two_cases(a: Nat) {
    a < Nat.2 implies (a = Nat.1 or a = Nat.0)
} by {
    if a < Nat.2 {
        lt_suc_right(a, Nat.1)
        a < Nat.2 implies a = Nat.1 or a < Nat.1
        a = Nat.1 or a < Nat.1
        if not a = Nat.1 {
            a < Nat.1
            lt_suc_right(a, Nat.0)
            a < Nat.1 implies a = Nat.0 or a < Nat.0
            a = Nat.0 or a < Nat.0
            not_lt_zero(a)
            not a < Nat.0
            a = Nat.0
            a = Nat.1 or a = Nat.0
        }
        a = Nat.1 or a = Nat.0
    }
}

/// A positive number below three is one or two.
theorem lt_three_cases(a: Nat) {
    a < Nat.3 implies (a = Nat.2 or a = Nat.1 or a = Nat.0)
} by {
    if a < Nat.3 {
        lt_suc_right(a, Nat.2)
        a < Nat.3 implies a = Nat.2 or a < Nat.2
        a = Nat.2 or a < Nat.2
        if not a = Nat.2 {
            a < Nat.2
            lt_two_cases(a)
            a = Nat.1 or a = Nat.0
            a = Nat.2 or a = Nat.1 or a = Nat.0
        }
        a = Nat.2 or a = Nat.1 or a = Nat.0
    }
}

/// A positive number below four is one, two or three.
theorem lt_four_cases(a: Nat) {
    a < Nat.4 implies (a = Nat.3 or a = Nat.2 or a = Nat.1 or a = Nat.0)
} by {
    if a < Nat.4 {
        lt_suc_right(a, Nat.3)
        a < Nat.4 implies a = Nat.3 or a < Nat.3
        a = Nat.3 or a < Nat.3
        if not a = Nat.3 {
            a < Nat.3
            lt_three_cases(a)
            a = Nat.2 or a = Nat.1 or a = Nat.0
            a = Nat.3 or a = Nat.2 or a = Nat.1 or a = Nat.0
        }
        a = Nat.3 or a = Nat.2 or a = Nat.1 or a = Nat.0
    }
}

/// The only two positive ordered parts summing to three are 2 and 1.
theorem two_parts_three(a: Nat, b: Nat) {
    Nat.1 <= b and b <= a and a + b = Nat.3 implies (a = Nat.2 and b = Nat.1)
} by {
    if Nat.1 <= b and b <= a and a + b = Nat.3 {
        Nat.1 <= b
        b <= a
        a + b = Nat.3
        lte_trans(Nat.1, b, a)
        Nat.1 <= a
        lte_add_left(a, Nat.1, b)
        Nat.1 <= b implies a + Nat.1 <= a + b
        a + Nat.1 <= a + b
        a + Nat.1 <= Nat.3
        add_one_right(a)
        a + Nat.1 = a.suc
        a.suc <= Nat.3
        lt_or_lte(a, Nat.2)
        if a < Nat.2 {
            lt_two_cases(a)
            a = Nat.1 or a = Nat.0
            if a = Nat.1 {
                Nat.1 + b = Nat.3
                Nat.3 = Nat.2 + Nat.1
                add_cancels_left(Nat.1, b, Nat.2)
                b = Nat.2
                b <= a
                Nat.2 <= Nat.1
                false
            }
            if a = Nat.0 {
                Nat.1 <= a
                Nat.1 <= Nat.0
                false
            }
            false
        }
        if not a < Nat.2 {
            lte_imp_not_lt(a, Nat.2)
            a <= Nat.2 implies not Nat.2 < a
            a.suc <= Nat.3
            lte_cancel_suc(a, Nat.2)
            a.suc <= Nat.2.suc implies a <= Nat.2
            a <= Nat.2
            lte_antisymm(a, Nat.2)
            a = Nat.2
            Nat.2 + b = Nat.3
            Nat.3 = Nat.2 + Nat.1
            add_cancels_left(Nat.2, b, Nat.1)
            b = Nat.1
            a = Nat.2 and b = Nat.1
        }
        a = Nat.2 and b = Nat.1
    }
}

/// When the larger part of a sum of four is below three, both parts are two.
theorem two_parts_four_small(a: Nat, b: Nat) {
    Nat.1 <= b and b <= a and a + b = Nat.4 and a < Nat.3 implies (a = Nat.2 and b = Nat.2)
} by {
    if Nat.1 <= b and b <= a and a + b = Nat.4 and a < Nat.3 {
        Nat.1 <= b
        b <= a
        a + b = Nat.4
        a < Nat.3
        lt_suc_right(a, Nat.2)
        a < Nat.3 implies a = Nat.2 or a < Nat.2
        a = Nat.2 or a < Nat.2
        if a = Nat.2 {
            Nat.2 + b = Nat.4
            Nat.4 = Nat.2 + Nat.2
            add_cancels_left(Nat.2, b, Nat.2)
            b = Nat.2
            a = Nat.2 and b = Nat.2
        }
        if not a = Nat.2 {
            a < Nat.2
            lt_suc_right(a, Nat.1)
            a < Nat.2 implies a = Nat.1 or a < Nat.1
            a = Nat.1 or a < Nat.1
            if a = Nat.1 {
                Nat.1 + b = Nat.4
                Nat.4 = Nat.1 + Nat.3
                add_cancels_left(Nat.1, b, Nat.3)
                b = Nat.3
                b <= a
                Nat.3 <= Nat.1
                false
            }
            if not a = Nat.1 {
                a < Nat.1
                lt_suc_right(a, Nat.0)
                a < Nat.1 implies a = Nat.0 or a < Nat.0
                a = Nat.0 or a < Nat.0
                not_lt_zero(a)
                not a < Nat.0
                a = Nat.0
                lte_trans(Nat.1, b, a)
                Nat.1 <= a
                Nat.1 <= Nat.0
                false
            }
            false
        }
        a = Nat.2 and b = Nat.2
    }
}

/// When the larger part of a sum of four is at least three, the parts are 3 and 1.
theorem two_parts_four_big(a: Nat, b: Nat) {
    Nat.1 <= b and b <= a and a + b = Nat.4 and not a < Nat.3 implies (a = Nat.3 and b = Nat.1)
} by {
    if Nat.1 <= b and b <= a and a + b = Nat.4 and not a < Nat.3 {
        Nat.1 <= b
        b <= a
        a + b = Nat.4
        not a < Nat.3
        lte_add_left(a, Nat.1, b)
        Nat.1 <= b implies a + Nat.1 <= a + b
        a + Nat.1 <= a + b
        a + Nat.1 <= Nat.4
        add_one_right(a)
        a + Nat.1 = a.suc
        a.suc <= Nat.4
        lte_cancel_suc(a, Nat.3)
        a.suc <= Nat.3.suc implies a <= Nat.3
        a <= Nat.3
        lte_imp_not_lt(a, Nat.3)
        a <= Nat.3 implies not Nat.3 < a
        lt_or_lte(Nat.3, a)
        Nat.3 < a or a <= Nat.3
        if Nat.3 < a {
            false
        }
        if not Nat.3 < a {
            a <= Nat.3
        }
        a <= Nat.3
        lte_antisymm(a, Nat.3)
        a = Nat.3
        Nat.3 + b = Nat.4
        Nat.4 = Nat.3 + Nat.1
        add_cancels_left(Nat.3, b, Nat.1)
        b = Nat.1
        a = Nat.3 and b = Nat.1
    }
}

/// The only two positive ordered parts summing to four are 3, 1 and 2, 2.
theorem two_parts_four(a: Nat, b: Nat) {
    Nat.1 <= b and b <= a and a + b = Nat.4 implies (a = Nat.3 and b = Nat.1) or (a = Nat.2 and b = Nat.2)

} by {
    if Nat.1 <= b and b <= a and a + b = Nat.4 {
        lt_or_lte(a, Nat.3)
        if a < Nat.3 {
            two_parts_four_small(a, b)
            Nat.1 <= b and b <= a and a + b = Nat.4 and a < Nat.3 implies (a = Nat.2 and b = Nat.2)
            a = Nat.2 and b = Nat.2
            or_intro_right(a = Nat.3 and b = Nat.1, a = Nat.2 and b = Nat.2)
            a = Nat.2 and b = Nat.2 implies (a = Nat.3 and b = Nat.1) or (a = Nat.2 and b = Nat.2)

            (a = Nat.3 and b = Nat.1) or (a = Nat.2 and b = Nat.2)
        }
        if not a < Nat.3 {
            two_parts_four_big(a, b)
            Nat.1 <= b and b <= a and a + b = Nat.4 and not a < Nat.3 implies (a = Nat.3 and b = Nat.1)
            a = Nat.3 and b = Nat.1
            or_intro_left(a = Nat.3 and b = Nat.1, a = Nat.2 and b = Nat.2)
            a = Nat.3 and b = Nat.1 implies (a = Nat.3 and b = Nat.1) or (a = Nat.2 and b = Nat.2)

            (a = Nat.3 and b = Nat.1) or (a = Nat.2 and b = Nat.2)
        }
        (a = Nat.3 and b = Nat.1) or (a = Nat.2 and b = Nat.2)
    }
}

/// The only three positive ordered parts summing to three are 1, 1, 1.
theorem three_parts_three(a: Nat, b: Nat, c: Nat) {
    Nat.1 <= a and Nat.1 <= b and Nat.1 <= c and b <= a and c <= b and
        a + b + c = Nat.3 implies a = Nat.1 and b = Nat.1 and c = Nat.1
} by {
    if Nat.1 <= a and Nat.1 <= b and Nat.1 <= c and b <= a and c <= b and
        a + b + c = Nat.3 {
        Nat.1 <= a
        Nat.1 <= b
        Nat.1 <= c
        b <= a
        c <= b
        a + b + c = Nat.3
        lte_add_left(b, Nat.1, c)
        Nat.1 <= c implies b + Nat.1 <= b + c
        b + Nat.1 <= b + c
        add_one_right(b)
        b + Nat.1 = b.suc
        b.suc <= b + c
        lte_add_left(a, b.suc, c)
        b.suc <= b + c implies a + b.suc <= a + (b + c)
        a + b.suc <= a + (b + c)
        add_one_right(a)
        a + Nat.1 = a.suc
        lte_add_left(a, Nat.1, b.suc)
        Nat.1 <= b.suc implies a + Nat.1 <= a + b.suc
        Nat.1 <= b.suc
        a + Nat.1 <= a + b.suc
        a.suc <= a + b.suc
        lte_trans(a.suc, a + b.suc, a + (b + c))
        a.suc <= a + (b + c)
        add_assoc(a, b, c)
        a + b + c = a + (b + c)
        a + (b + c) = Nat.3
        a.suc <= Nat.3
        lt_or_lte(a, Nat.1)
        if a < Nat.1 {
            a = Nat.0
            Nat.1 <= a
            Nat.1 <= Nat.0
            false
        }
        if not a < Nat.1 {
            lte_add_right(Nat.1, b, Nat.1)
            Nat.1 <= b implies Nat.1 + Nat.1 <= b + Nat.1
            Nat.1 + Nat.1 = Nat.2
            Nat.2 <= b + Nat.1
            add_one_right(b)
            b + Nat.1 = b.suc
            Nat.2 <= b.suc
            lte_trans(Nat.2, b.suc, b + c)
            Nat.2 <= b + c
            lte_add_left(a, Nat.2, b + c)
            Nat.2 <= b + c implies a + Nat.2 <= a + (b + c)
            a + Nat.2 <= a + (b + c)
            a + (b + c) = Nat.3
            a + Nat.2 <= Nat.3
            add_one_right(a)
            a + Nat.1 = a.suc
            add_one_right(a.suc)
            a.suc + Nat.1 = a.suc.suc
            a + Nat.2 = a.suc.suc
            a.suc.suc <= Nat.3
            lte_cancel_suc(a.suc, Nat.2)
            a.suc.suc <= Nat.2.suc implies a.suc <= Nat.2
            a.suc <= Nat.2
            lte_cancel_suc(a, Nat.1)
            a.suc <= Nat.1.suc implies a <= Nat.1
            a <= Nat.1
            lte_antisymm(a, Nat.1)
            Nat.1 <= a and a <= Nat.1
            a = Nat.1
            a + (b + c) = Nat.3
            add_cancels_left(Nat.1, b + c, Nat.2)
            Nat.1 + (b + c) = Nat.3
            add_cancels_left(Nat.1, b + c, Nat.2)
            b + c = Nat.2
            lt_or_lte(b, Nat.1)
            if b < Nat.1 {
                b = Nat.0
                Nat.1 <= b
                Nat.1 <= Nat.0
                false
            }
            if not b < Nat.1 {
                lte_imp_not_lt(b, Nat.1)
                b <= Nat.1 implies not Nat.1 < b
                lte_add_left(b, Nat.1, c)
                Nat.1 <= c implies b + Nat.1 <= b + c
                b + Nat.1 <= b + c
                add_one_right(b)
                b + Nat.1 = b.suc
                b.suc <= b + c
                b.suc <= Nat.2
                lte_cancel_suc(b, Nat.1)
                b.suc <= Nat.1.suc implies b <= Nat.1
                b <= Nat.1
                lte_antisymm(b, Nat.1)
                b = Nat.1
                Nat.1 + c = Nat.2
                add_cancels_left(Nat.1, c, Nat.1)
                c = Nat.1
                a = Nat.1 and b = Nat.1 and c = Nat.1
            }
            a = Nat.1 and b = Nat.1 and c = Nat.1
        }
        a = Nat.1 and b = Nat.1 and c = Nat.1
    }
}

/// The only three positive ordered parts summing to four are 2, 1, 1.
theorem three_parts_four(a: Nat, b: Nat, c: Nat) {
    Nat.1 <= a and Nat.1 <= b and Nat.1 <= c and b <= a and c <= b and
        a + b + c = Nat.4 implies a = Nat.2 and b = Nat.1 and c = Nat.1
} by {
    if Nat.1 <= a and Nat.1 <= b and Nat.1 <= c and b <= a and c <= b and
        a + b + c = Nat.4 {
        Nat.1 <= a
        Nat.1 <= b
        Nat.1 <= c
        b <= a
        c <= b
        a + b + c = Nat.4
        add_assoc(a, b, c)
        a + b + c = a + (b + c)
        a + (b + c) = Nat.4
        lt_or_lte(a, Nat.2)
        if a < Nat.2 {
            lt_two_cases(a)
            a = Nat.1 or a = Nat.0
            if a = Nat.1 {
                Nat.1 + (b + c) = Nat.4
                add_cancels_left(Nat.1, b + c, Nat.3)
                b + c = Nat.3
                lt_or_lte(b, Nat.2)
                if b < Nat.2 {
                    lt_two_cases(b)
                    b = Nat.1 or b = Nat.0
                    if b = Nat.1 {
                        Nat.1 + c = Nat.3
                        add_cancels_left(Nat.1, c, Nat.2)
                        c = Nat.2
                        c <= b
                        Nat.2 <= Nat.1
                        false
                    }
                    if b = Nat.0 {
                        Nat.1 <= b
                        Nat.1 <= Nat.0
                        false
                    }
                    false
                }
                if not b < Nat.2 {
                    lte_imp_not_lt(b, Nat.2)
                    b <= Nat.2 implies not Nat.2 < b
                    lte_add_left(b, Nat.1, c)
                    Nat.1 <= c implies b + Nat.1 <= b + c
                    b + Nat.1 <= b + c
                    add_one_right(b)
                    b + Nat.1 = b.suc
                    b.suc <= b + c
                    b.suc <= Nat.3
                    lte_cancel_suc(b, Nat.2)
                    b.suc <= Nat.2.suc implies b <= Nat.2
                    b <= Nat.2
                    lte_antisymm(b, Nat.2)
                    b = Nat.2
                    Nat.2 + c = Nat.3
                    add_cancels_left(Nat.2, c, Nat.1)
                    c = Nat.1
                    c <= b
                    Nat.1 <= Nat.2
                    b <= a
                    Nat.2 <= Nat.1
                    false
                }
                false
            }
            if a = Nat.0 {
                Nat.1 <= a
                Nat.1 <= Nat.0
                false
            }
            false
        }
        if not a < Nat.2 {
            lte_imp_not_lt(a, Nat.2)
            a <= Nat.2 implies not Nat.2 < a
            a + (b + c) = Nat.4
            lte_add_left(a, Nat.2, b + c)
            Nat.2 <= b + c implies a + Nat.2 <= a + (b + c)
            lte_add_left(b, Nat.1, c)
            Nat.1 <= c implies b + Nat.1 <= b + c
            add_one_right(b)
            b + Nat.1 = b.suc
            lte_add_right(Nat.1, b, Nat.1)
            Nat.1 <= b implies Nat.1 + Nat.1 <= b + Nat.1
            Nat.1 + Nat.1 = Nat.2
            Nat.2 <= b + Nat.1
            add_one_right(b)
            b + Nat.1 = b.suc
            Nat.2 <= b.suc
            lte_trans(Nat.2, b.suc, b + c)
            Nat.2 <= b + c
            a + Nat.2 <= a + (b + c)
            a + Nat.2 <= Nat.4
            add_one_right(a)
            a + Nat.1 = a.suc
            add_one_right(a.suc)
            a.suc + Nat.1 = a.suc.suc
            a + Nat.2 = a.suc.suc
            a.suc.suc <= Nat.4
            lte_cancel_suc(a, Nat.3)
            a.suc <= Nat.3
            lte_cancel_suc(a, Nat.2)
            a <= Nat.2
            lte_antisymm(a, Nat.2)
            a = Nat.2
            Nat.2 + (b + c) = Nat.4
            add_cancels_left(Nat.2, b + c, Nat.2)
            b + c = Nat.2
            lt_or_lte(b, Nat.1)
            if b < Nat.1 {
                b = Nat.0
                Nat.1 <= b
                Nat.1 <= Nat.0
                false
            }
            if not b < Nat.1 {
                lte_imp_not_lt(b, Nat.1)
                b <= Nat.1 implies not Nat.1 < b
                lte_add_left(b, Nat.1, c)
                Nat.1 <= c implies b + Nat.1 <= b + c
                b + Nat.1 <= b + c
                add_one_right(b)
                b + Nat.1 = b.suc
                b.suc <= b + c
                b.suc <= Nat.2
                lte_cancel_suc(b, Nat.1)
                b.suc <= Nat.1.suc implies b <= Nat.1
                b <= Nat.1
                lte_antisymm(b, Nat.1)
                b = Nat.1
                Nat.1 + c = Nat.2
                add_cancels_left(Nat.1, c, Nat.1)
                c = Nat.1
                a = Nat.2 and b = Nat.1 and c = Nat.1
            }
            a = Nat.2 and b = Nat.1 and c = Nat.1
        }
        a = Nat.2 and b = Nat.1 and c = Nat.1
    }
}

/// The partitions of two are exactly 2 and 1 + 1.
theorem partition_two_members(l: List[Nat]) {
    is_partition(l, Nat.2) implies (l = List.cons(Nat.2, List.nil[Nat]) or

         l = List.cons(Nat.1, List.cons(Nat.1, List.nil[Nat])))
} by {
    if is_partition(l, Nat.2) {
        is_partition_sum(l, Nat.2)
        sum(l) = Nat.2
        match l {
            List.nil {
                sum(List.nil[Nat]) = Nat.0
                Nat.0 = Nat.2
                false
            }
            List.cons(a, t) {
                match t {
                    List.nil {
                        sum(List.cons(a, List.nil[Nat])) = a + sum(List.nil[Nat])
                        sum(List.nil[Nat]) = Nat.0
                        a + Nat.0 = a
                        sum(List.cons(a, List.nil[Nat])) = a
                        a = Nat.2
                        l = List.cons(a, List.nil[Nat])
                        l = List.cons(Nat.2, List.nil[Nat])
                    }
                    List.cons(b, t2) {
                        match t2 {
                            List.nil {
                                sum(List.cons(a, List.cons(b, List.nil[Nat]))) =
                                    a + sum(List.cons(b, List.nil[Nat]))
                                sum(List.cons(b, List.nil[Nat])) = b + sum(List.nil[Nat])
                                sum(List.nil[Nat]) = Nat.0
                                b + Nat.0 = b
                                sum(List.cons(b, List.nil[Nat])) = b
                                a + b = Nat.2
                                is_partition_all_positive(l, Nat.2)
                                all_positive(List.cons(a, List.cons(b, List.nil[Nat])))
                                all_positive_head(a, List.cons(b, List.nil[Nat]))
                                Nat.0 < a
                                lt_imp_lte_suc(Nat.0, a)
                                Nat.1 <= a
                                all_positive_tail(a, List.cons(b, List.nil[Nat]))
                                all_positive(List.cons(b, List.nil[Nat]))
                                all_positive_head(b, List.nil[Nat])
                                Nat.0 < b
                                lt_imp_lte_suc(Nat.0, b)
                                Nat.1 <= b
                                lte_add_left(a, Nat.1, b)
                                Nat.1 <= b implies a + Nat.1 <= a + b
                                a + Nat.1 <= a + b
                                a + b = Nat.2
                                a + Nat.1 <= Nat.2
                                add_one_right(a)
                                a + Nat.1 = a.suc
                                lte_cancel_suc(a, Nat.1)
                                a.suc <= Nat.1.suc implies a <= Nat.1
                                a <= Nat.1
                                lte_antisymm(a, Nat.1)
                                Nat.1 <= a and a <= Nat.1
                                a = Nat.1
                                add_comm(b, Nat.1)
                                b + Nat.1 = Nat.1 + b
                                add_one_right(b)
                                b + Nat.1 = b.suc
                                Nat.1 + b = b.suc
                                Nat.1 + b = Nat.2
                                one_plus_one
                                Nat.1 + Nat.1 = Nat.2
                                add_cancels_left(Nat.1, b, Nat.1)
                                Nat.1 + b = Nat.1 + Nat.1 implies b = Nat.1
                                b = Nat.1
                                l = List.cons(Nat.1, List.cons(Nat.1, List.nil[Nat]))
                            }
                            List.cons(c, t3) {
                                sum(List.cons(a, List.cons(b, List.cons(c, t3)))) =
                                    a + sum(List.cons(b, List.cons(c, t3)))
                                sum(List.cons(b, List.cons(c, t3))) = b + sum(List.cons(c, t3))
                                sum(List.cons(c, t3)) = c + sum(t3)
                                a + (b + (c + sum(t3))) = Nat.2
                                is_partition_all_positive(l, Nat.2)
                                all_positive(List.cons(a, List.cons(b, List.cons(c, t3))))
                                all_positive_head(a, List.cons(b, List.cons(c, t3)))
                                Nat.0 < a
                                lt_imp_lte_suc(Nat.0, a)
                                Nat.1 <= a
                                all_positive_tail(a, List.cons(b, List.cons(c, t3)))
                                all_positive(List.cons(b, List.cons(c, t3)))
                                all_positive_head(b, List.cons(c, t3))
                                Nat.0 < b
                                lt_imp_lte_suc(Nat.0, b)
                                Nat.1 <= b
                                all_positive_tail(b, List.cons(c, t3))
                                all_positive(List.cons(c, t3))
                                all_positive_head(c, t3)
                                Nat.0 < c
                                lt_imp_lte_suc(Nat.0, c)
                                Nat.1 <= c
                                lte_add_right(Nat.1, Nat.1, b)
                                Nat.1 + Nat.1 <= b + Nat.1
                                Nat.1 + Nat.1 = Nat.2
                                Nat.2 <= b + Nat.1
                                add_one_right(b)
                                b + Nat.1 = b.suc
                                Nat.2 <= b.suc
                                lte_cancel_suc(Nat.1, b)
                                Nat.1.suc <= b.suc implies Nat.1 <= b
                                Nat.1 <= b
                                zero_lte(c + sum(t3))
                                Nat.0 <= c + sum(t3)
                                all_positive_tail(b, List.cons(c, t3))
                                all_positive(List.cons(c, t3))
                                all_positive_head(c, t3)
                                Nat.0 < c
                                lt_imp_lte_suc(Nat.0, c)
                                Nat.1 <= c
                                lte_add_left(c, Nat.0, sum(t3))
                                Nat.0 <= sum(t3) implies c + Nat.0 <= c + sum(t3)
                                zero_lte(sum(t3))
                                Nat.0 <= sum(t3)
                                c + Nat.0 <= c + sum(t3)
                                c + Nat.0 = c
                                c <= c + sum(t3)
                                lte_trans(Nat.1, c, c + sum(t3))
                                Nat.1 <= c + sum(t3)
                                lte_add_left(b, Nat.1, c + sum(t3))
                                Nat.1 <= c + sum(t3) implies b + Nat.1 <= b + (c + sum(t3))
                                b + Nat.1 <= b + (c + sum(t3))
                                lte_trans(Nat.2, b + Nat.1, b + (c + sum(t3)))
                                Nat.2 <= b + (c + sum(t3))
                                lte_add_left(a, Nat.2, b + (c + sum(t3)))
                                Nat.2 <= b + (c + sum(t3)) implies a + Nat.2 <= a + (b + (c + sum(t3)))
                                a + Nat.2 <= a + (b + (c + sum(t3)))
                                a + (b + (c + sum(t3))) = Nat.2
                                a + Nat.2 <= Nat.2
                                add_comm(Nat.2, a)
                                Nat.2 + a = a + Nat.2
                                lte_add_left(Nat.2, Nat.0, a)
                                Nat.0 <= a implies Nat.2 + Nat.0 <= Nat.2 + a
                                zero_lte(a)
                                Nat.0 <= a
                                Nat.2 + Nat.0 <= Nat.2 + a
                                Nat.2 + Nat.0 = Nat.2
                                Nat.2 <= Nat.2 + a
                                Nat.2 <= a + Nat.2
                                lte_antisymm(a + Nat.2, Nat.2)
                                a + Nat.2 <= Nat.2 and Nat.2 <= a + Nat.2
                                a + Nat.2 = Nat.2
                                add_comm(a, Nat.2)
                                a + Nat.2 = Nat.2 + a
                                Nat.2 + a = Nat.2
                                add_zero_left(Nat.2)
                                Nat.0 + Nat.2 = Nat.2
                                add_cancels_left(Nat.2, a, Nat.0)
                                Nat.2 + a = Nat.2 + Nat.0 implies a = Nat.0
                                Nat.2 + Nat.0 = Nat.2
                                a = Nat.0
                                Nat.0 < a
                                lt_not_ref(Nat.0)
                                not Nat.0 < Nat.0
                                false
                            }
                        }
                    }
                }
            }
        }
        l = List.cons(Nat.2, List.nil[Nat]) or
            l = List.cons(Nat.1, List.cons(Nat.1, List.nil[Nat]))
    }
}

/// The lists 2 and 1 + 1 are the only partitions of two.
theorem partition_two_sound(l: List[Nat]) {
    (l = List.cons(Nat.2, List.nil[Nat]) or
     l = List.cons(Nat.1, List.cons(Nat.1, List.nil[Nat])))
    implies is_partition(l, Nat.2)
} by {
    if l = List.cons(Nat.2, List.nil[Nat]) or
        l = List.cons(Nat.1, List.cons(Nat.1, List.nil[Nat])) {
        if l = List.cons(Nat.2, List.nil[Nat]) {
            sum(List.cons(Nat.2, List.nil[Nat])) = Nat.2 + sum(List.nil[Nat])
            sum(List.nil[Nat]) = Nat.0
            Nat.2 + Nat.0 = Nat.2
            sum(List.cons(Nat.2, List.nil[Nat])) = Nat.2
            non_increasing(List.cons(Nat.2, List.nil[Nat])) =
                non_increasing_from(Nat.2, List.nil[Nat])
            non_increasing_from(Nat.2, List.nil[Nat]) = true
            non_increasing(List.cons(Nat.2, List.nil[Nat])) = true
            Nat.0 < Nat.2
            ap_single(Nat.2)
            Nat.0 < Nat.2 implies all_positive(List.cons(Nat.2, List.nil[Nat])) = true
            all_positive(List.cons(Nat.2, List.nil[Nat])) = true
            is_partition_intro(List.cons(Nat.2, List.nil[Nat]), Nat.2)
            is_partition(List.cons(Nat.2, List.nil[Nat]), Nat.2)
            is_partition(l, Nat.2)
        }
        if l = List.cons(Nat.1, List.cons(Nat.1, List.nil[Nat])) {
            sum(List.cons(Nat.1, List.cons(Nat.1, List.nil[Nat]))) =
                Nat.1 + sum(List.cons(Nat.1, List.nil[Nat]))
            sum(List.cons(Nat.1, List.nil[Nat])) = Nat.1 + sum(List.nil[Nat])
            sum(List.nil[Nat]) = Nat.0
            Nat.1 + Nat.0 = Nat.1
            sum(List.cons(Nat.1, List.nil[Nat])) = Nat.1
            Nat.1 + Nat.1 = Nat.2
            sum(List.cons(Nat.1, List.cons(Nat.1, List.nil[Nat]))) = Nat.2
            all_positive(List.cons(Nat.1, List.nil[Nat])) =
                (Nat.0 < Nat.1 and all_positive(List.nil[Nat]))
            all_positive(List.nil[Nat]) = true
            Nat.0 < Nat.1
            all_positive(List.cons(Nat.1, List.nil[Nat])) = true
            Nat.0 < Nat.1
            all_positive(List.cons(Nat.1, List.cons(Nat.1, List.nil[Nat]))) =
                (Nat.0 < Nat.1 and all_positive(List.cons(Nat.1, List.nil[Nat])))
            all_positive(List.cons(Nat.1, List.cons(Nat.1, List.nil[Nat]))) = true
            non_increasing(List.cons(Nat.1, List.cons(Nat.1, List.nil[Nat]))) =
                non_increasing_from(Nat.1, List.cons(Nat.1, List.nil[Nat]))
            non_increasing_from(Nat.1, List.cons(Nat.1, List.nil[Nat])) =
                (Nat.1 <= Nat.1 and non_increasing_from(Nat.1, List.nil[Nat]))
            non_increasing_from(Nat.1, List.nil[Nat]) = true
            lte_ref(Nat.1)
            Nat.1 <= Nat.1
            non_increasing(List.cons(Nat.1, List.cons(Nat.1, List.nil[Nat]))) = true
            is_partition_intro(List.cons(Nat.1, List.cons(Nat.1, List.nil[Nat])), Nat.2)
            is_partition(List.cons(Nat.1, List.cons(Nat.1, List.nil[Nat])), Nat.2)
            is_partition(l, Nat.2)
        }
        is_partition(l, Nat.2)
    }
}

/// The partitions of two are exactly 2 and 1 + 1.
theorem partition_two_char(l: List[Nat]) {
    is_partition(l, Nat.2) =
        (l = List.cons(Nat.2, List.nil[Nat]) or
         l = List.cons(Nat.1, List.cons(Nat.1, List.nil[Nat])))
} by {
    partition_two_members(l)
    is_partition(l, Nat.2) implies (l = List.cons(Nat.2, List.nil[Nat]) or l = List.cons(Nat.1, List.cons(Nat.1, List.nil[Nat])))
    partition_two_sound(l)
    (l = List.cons(Nat.2, List.nil[Nat]) or l = List.cons(Nat.1, List.cons(Nat.1, List.nil[Nat]))) implies is_partition(l, Nat.2)
    (is_partition(l, Nat.2) implies (l = List.cons(Nat.2, List.nil[Nat]) or l = List.cons(Nat.1, List.cons(Nat.1, List.nil[Nat])))) and
        ((l = List.cons(Nat.2, List.nil[Nat]) or l = List.cons(Nat.1, List.cons(Nat.1, List.nil[Nat]))) implies is_partition(l, Nat.2))
    bool_eq_of_iff(is_partition(l, Nat.2), l = List.cons(Nat.2, List.nil[Nat]) or l = List.cons(Nat.1, List.cons(Nat.1, List.nil[Nat])))
    is_partition(l, Nat.2) = (l = List.cons(Nat.2, List.nil[Nat]) or l = List.cons(Nat.1, List.cons(Nat.1, List.nil[Nat])))
}

// ---------------------------------------------------------------------------
// Concrete evaluations used by the sound proofs below.
// ---------------------------------------------------------------------------

/// The sum of a list of positive elements is at least its length.
theorem sum_ge_length_all_positive(l: List[Nat]) {
    all_positive(l) implies l.length <= sum(l)
} by {
    define p(xs: List[Nat]) -> Bool {
        all_positive(xs) implies xs.length <= sum(xs)
    }
    if all_positive(List.nil[Nat]) {
        List.nil[Nat].length = Nat.0
        sum(List.nil[Nat]) = Nat.0
        lte_ref(Nat.0)
        Nat.0 <= Nat.0
    }
    p(List.nil[Nat])
    forall(head: Nat, tail: List[Nat]) {
        if p(tail) {
            if all_positive(List.cons(head, tail)) {
                all_positive_head(head, tail)
                Nat.0 < head
                lt_imp_lte_suc(Nat.0, head)
                Nat.1 <= head
                all_positive_tail(head, tail)
                all_positive(tail)
                p(tail)
                all_positive(tail) implies tail.length <= sum(tail)
                tail.length <= sum(tail)
                lte_add_right(Nat.1, tail.length, sum(tail))
                tail.length <= sum(tail) implies tail.length + Nat.1 <= sum(tail) + Nat.1
                tail.length + Nat.1 <= sum(tail) + Nat.1
                lte_add_left(sum(tail), Nat.1, head)
                Nat.1 <= head implies sum(tail) + Nat.1 <= sum(tail) + head
                sum(tail) + Nat.1 <= sum(tail) + head
                lte_trans(tail.length + Nat.1, sum(tail) + Nat.1, sum(tail) + head)
                tail.length + Nat.1 <= sum(tail) + head
                add_one_right(tail.length)
                tail.length + Nat.1 = tail.length.suc
                tail.length.suc <= sum(tail) + head
                add_comm(sum(tail), head)
                sum(tail) + head = head + sum(tail)
                tail.length.suc <= head + sum(tail)
                List.cons(head, tail).length = tail.length.suc
                sum(List.cons(head, tail)) = head + sum(tail)
                List.cons(head, tail).length <= sum(List.cons(head, tail))
            }
            p(List.cons(head, tail))
        }
    }
    List.induction(function(xs: List[Nat]) { p(xs) })
    forall(xs: List[Nat]) { p(xs) }
    p(l)
}



/// The list 2, 1 is non-increasing.
theorem ni_21 {
    non_increasing(List.cons(Nat.2, List.cons(Nat.1, List.nil[Nat]))) = true
} by {
    non_increasing(List.cons(Nat.2, List.cons(Nat.1, List.nil[Nat]))) =
        non_increasing_from(Nat.2, List.cons(Nat.1, List.nil[Nat]))
    non_increasing_from(Nat.1, List.nil[Nat]) = true
    Nat.1 + Nat.1 = Nat.2
    exists(c: Nat) { Nat.1 + c = Nat.2 }
    Nat.1 <= Nat.2
    non_increasing_from(Nat.2, List.cons(Nat.1, List.nil[Nat])) = true
    non_increasing(List.cons(Nat.2, List.cons(Nat.1, List.nil[Nat]))) = true
}

/// The list 3, 1 is non-increasing.
theorem ni_31 {
    non_increasing(List.cons(Nat.3, List.cons(Nat.1, List.nil[Nat]))) = true
} by {
    non_increasing(List.cons(Nat.3, List.cons(Nat.1, List.nil[Nat]))) =
        non_increasing_from(Nat.3, List.cons(Nat.1, List.nil[Nat]))
    non_increasing_from(Nat.1, List.nil[Nat]) = true
    Nat.1 + Nat.2 = Nat.3
    exists(c: Nat) { Nat.1 + c = Nat.3 }
    Nat.1 <= Nat.3
    non_increasing_from(Nat.3, List.cons(Nat.1, List.nil[Nat])) = true
    non_increasing(List.cons(Nat.3, List.cons(Nat.1, List.nil[Nat]))) = true
}

/// The list 2, 2 is non-increasing.
theorem ni_22 {
    non_increasing(List.cons(Nat.2, List.cons(Nat.2, List.nil[Nat]))) = true
} by {
    non_increasing(List.cons(Nat.2, List.cons(Nat.2, List.nil[Nat]))) =
        non_increasing_from(Nat.2, List.cons(Nat.2, List.nil[Nat]))
    non_increasing_from(Nat.2, List.nil[Nat]) = true
    lte_ref(Nat.2)
    Nat.2 <= Nat.2
    non_increasing_from(Nat.2, List.cons(Nat.2, List.nil[Nat])) = true
    non_increasing(List.cons(Nat.2, List.cons(Nat.2, List.nil[Nat]))) = true
}

/// The list 1, 1, 1 is non-increasing.
theorem ni_111 {
    non_increasing(List.cons(Nat.1, List.cons(Nat.1, List.cons(Nat.1, List.nil[Nat])))) = true
} by {
    non_increasing(List.cons(Nat.1, List.cons(Nat.1, List.cons(Nat.1, List.nil[Nat])))) =
        non_increasing_from(Nat.1, List.cons(Nat.1, List.cons(Nat.1, List.nil[Nat])))
    non_increasing_from(Nat.1, List.cons(Nat.1, List.nil[Nat])) =
        (Nat.1 <= Nat.1 and non_increasing_from(Nat.1, List.nil[Nat]))
    non_increasing_from(Nat.1, List.nil[Nat]) = true
    lte_ref(Nat.1)
    Nat.1 <= Nat.1
    non_increasing_from(Nat.1, List.cons(Nat.1, List.nil[Nat])) = true
    lte_ref(Nat.1)
    Nat.1 <= Nat.1
    non_increasing_from(Nat.1, List.cons(Nat.1, List.cons(Nat.1, List.nil[Nat]))) =
        (Nat.1 <= Nat.1 and non_increasing_from(Nat.1, List.cons(Nat.1, List.nil[Nat])))
    non_increasing_from(Nat.1, List.cons(Nat.1, List.cons(Nat.1, List.nil[Nat]))) = true
    non_increasing(List.cons(Nat.1, List.cons(Nat.1, List.cons(Nat.1, List.nil[Nat])))) = true
}

/// The list 2, 1, 1 is non-increasing.
theorem ni_211 {
    non_increasing(List.cons(Nat.2, List.cons(Nat.1, List.cons(Nat.1, List.nil[Nat])))) = true
} by {
    non_increasing(List.cons(Nat.2, List.cons(Nat.1, List.cons(Nat.1, List.nil[Nat])))) =
        non_increasing_from(Nat.2, List.cons(Nat.1, List.cons(Nat.1, List.nil[Nat])))
    non_increasing_from(Nat.1, List.cons(Nat.1, List.nil[Nat])) =
        (Nat.1 <= Nat.1 and non_increasing_from(Nat.1, List.nil[Nat]))
    non_increasing_from(Nat.1, List.nil[Nat]) = true
    lte_ref(Nat.1)
    Nat.1 <= Nat.1
    non_increasing_from(Nat.1, List.cons(Nat.1, List.nil[Nat])) = true
    lte_ref(Nat.1)
    Nat.1 <= Nat.1
    non_increasing_from(Nat.1, List.cons(Nat.1, List.cons(Nat.1, List.nil[Nat]))) =
        (Nat.1 <= Nat.1 and non_increasing_from(Nat.1, List.cons(Nat.1, List.nil[Nat])))
    non_increasing_from(Nat.1, List.cons(Nat.1, List.cons(Nat.1, List.nil[Nat]))) = true
    lte_ref(Nat.2)
    Nat.2 <= Nat.2
    non_increasing_from(Nat.2, List.cons(Nat.1, List.cons(Nat.1, List.nil[Nat]))) =
        (Nat.1 <= Nat.2 and non_increasing_from(Nat.1, List.cons(Nat.1, List.nil[Nat])))
    Nat.1 <= Nat.2
    non_increasing_from(Nat.2, List.cons(Nat.1, List.cons(Nat.1, List.nil[Nat]))) = true
    non_increasing(List.cons(Nat.2, List.cons(Nat.1, List.cons(Nat.1, List.nil[Nat])))) = true
}

/// The list 1, 1, 1, 1 is non-increasing.
theorem ni_1111 {
    non_increasing(List.cons(Nat.1, List.cons(Nat.1, List.cons(Nat.1, List.cons(Nat.1, List.nil[Nat]))))) = true
} by {
    non_increasing(List.cons(Nat.1, List.cons(Nat.1, List.cons(Nat.1, List.cons(Nat.1, List.nil[Nat]))))) =
        non_increasing_from(Nat.1, List.cons(Nat.1, List.cons(Nat.1, List.cons(Nat.1, List.nil[Nat]))))
    non_increasing_from(Nat.1, List.cons(Nat.1, List.nil[Nat])) =
        (Nat.1 <= Nat.1 and non_increasing_from(Nat.1, List.nil[Nat]))
    non_increasing_from(Nat.1, List.nil[Nat]) = true
    lte_ref(Nat.1)
    Nat.1 <= Nat.1
    non_increasing_from(Nat.1, List.cons(Nat.1, List.nil[Nat])) = true
    lte_ref(Nat.1)
    Nat.1 <= Nat.1
    non_increasing_from(Nat.1, List.cons(Nat.1, List.cons(Nat.1, List.nil[Nat]))) =
        (Nat.1 <= Nat.1 and non_increasing_from(Nat.1, List.cons(Nat.1, List.nil[Nat])))
    non_increasing_from(Nat.1, List.cons(Nat.1, List.cons(Nat.1, List.nil[Nat]))) = true
    lte_ref(Nat.1)
    Nat.1 <= Nat.1
    non_increasing_from(Nat.1, List.cons(Nat.1, List.cons(Nat.1, List.cons(Nat.1, List.nil[Nat])))) =
        (Nat.1 <= Nat.1 and non_increasing_from(Nat.1, List.cons(Nat.1, List.cons(Nat.1, List.nil[Nat]))))
    non_increasing_from(Nat.1, List.cons(Nat.1, List.cons(Nat.1, List.cons(Nat.1, List.nil[Nat])))) = true
    non_increasing(List.cons(Nat.1, List.cons(Nat.1, List.cons(Nat.1, List.cons(Nat.1, List.nil[Nat]))))) = true
}

/// The list 2, 1 is all-positive.
theorem ap_21 {
    all_positive(List.cons(Nat.2, List.cons(Nat.1, List.nil[Nat]))) = true
} by {
    all_positive(List.cons(Nat.1, List.nil[Nat])) = true
    Nat.0 < Nat.2
    all_positive(List.cons(Nat.2, List.cons(Nat.1, List.nil[Nat]))) = true
}

/// The list 3, 1 is all-positive.
theorem ap_31 {
    all_positive(List.cons(Nat.3, List.cons(Nat.1, List.nil[Nat]))) = true
} by {
    all_positive(List.cons(Nat.1, List.nil[Nat])) = true
    Nat.0 < Nat.3
    all_positive(List.cons(Nat.3, List.cons(Nat.1, List.nil[Nat]))) = true
}

/// The list 2, 2 is all-positive.
theorem ap_22 {
    all_positive(List.cons(Nat.2, List.cons(Nat.2, List.nil[Nat]))) = true
} by {
    Nat.0 < Nat.2
    ap_single(Nat.2)
    Nat.0 < Nat.2 implies all_positive(List.cons(Nat.2, List.nil[Nat])) = true
    all_positive(List.cons(Nat.2, List.nil[Nat])) = true
    Nat.0 < Nat.2
    all_positive(List.cons(Nat.2, List.cons(Nat.2, List.nil[Nat]))) = true
}

/// The list 1, 1, 1 is all-positive.
theorem ap_111 {
    all_positive(List.cons(Nat.1, List.cons(Nat.1, List.cons(Nat.1, List.nil[Nat])))) = true
} by {
    all_positive(List.cons(Nat.1, List.nil[Nat])) = true
    Nat.0 < Nat.1
    all_positive(List.cons(Nat.1, List.cons(Nat.1, List.nil[Nat]))) = true
    Nat.0 < Nat.1
    all_positive(List.cons(Nat.1, List.cons(Nat.1, List.cons(Nat.1, List.nil[Nat])))) = true
}

/// The list 2, 1, 1 is all-positive.
theorem ap_211 {
    all_positive(List.cons(Nat.2, List.cons(Nat.1, List.cons(Nat.1, List.nil[Nat])))) = true
} by {
    all_positive(List.cons(Nat.1, List.nil[Nat])) = true
    Nat.0 < Nat.1
    all_positive(List.cons(Nat.1, List.cons(Nat.1, List.nil[Nat]))) = true
    Nat.0 < Nat.2
    all_positive(List.cons(Nat.2, List.cons(Nat.1, List.cons(Nat.1, List.nil[Nat])))) = true
}

/// The list 1, 1, 1, 1 is all-positive.
theorem ap_1111 {
    all_positive(List.cons(Nat.1, List.cons(Nat.1, List.cons(Nat.1, List.cons(Nat.1, List.nil[Nat]))))) = true
} by {
    all_positive(List.cons(Nat.1, List.nil[Nat])) = true
    Nat.0 < Nat.1
    all_positive(List.cons(Nat.1, List.cons(Nat.1, List.nil[Nat]))) = true
    Nat.0 < Nat.1
    all_positive(List.cons(Nat.1, List.cons(Nat.1, List.cons(Nat.1, List.nil[Nat])))) = true
    Nat.0 < Nat.1
    all_positive(List.cons(Nat.1, List.cons(Nat.1, List.cons(Nat.1, List.cons(Nat.1, List.nil[Nat]))))) = true
}

// ---------------------------------------------------------------------------
// The partitions of three and four, by explicit enumeration.
// ---------------------------------------------------------------------------

/// The partitions of three are exactly 3, 2 + 1 and 1 + 1 + 1.
theorem partition_three_members(l: List[Nat]) {
    is_partition(l, Nat.3) implies (l = List.cons(Nat.3, List.nil[Nat]) or
         l = List.cons(Nat.2, List.cons(Nat.1, List.nil[Nat])) or
         l = List.cons(Nat.1, List.cons(Nat.1, List.cons(Nat.1, List.nil[Nat]))))
} by {
    if is_partition(l, Nat.3) {
        is_partition_sum(l, Nat.3)
        sum(l) = Nat.3
        match l {
            List.nil {
                sum(List.nil[Nat]) = Nat.0
                Nat.0 = Nat.3
                false
            }
            List.cons(a, t) {
                match t {
                    List.nil {
                        sum(List.cons(a, List.nil[Nat])) = a + sum(List.nil[Nat])
                        sum(List.nil[Nat]) = Nat.0
                        a + Nat.0 = a
                        sum(List.cons(a, List.nil[Nat])) = a
                        a = Nat.3
                        l = List.cons(a, List.nil[Nat])
                        l = List.cons(Nat.3, List.nil[Nat])
                    }
                    List.cons(b, t2) {
                        match t2 {
                            List.nil {
                                sum(List.cons(a, List.cons(b, List.nil[Nat]))) =
                                    a + sum(List.cons(b, List.nil[Nat]))
                                sum(List.cons(b, List.nil[Nat])) = b + sum(List.nil[Nat])
                                sum(List.nil[Nat]) = Nat.0
                                b + Nat.0 = b
                                sum(List.cons(b, List.nil[Nat])) = b
                                a + b = Nat.3
                                is_partition_all_positive(l, Nat.3)
                                all_positive(List.cons(a, List.cons(b, List.nil[Nat])))
                                all_positive_head(a, List.cons(b, List.nil[Nat]))
                                Nat.0 < a
                                lt_imp_lte_suc(Nat.0, a)
                                Nat.1 <= a
                                all_positive_tail(a, List.cons(b, List.nil[Nat]))
                                all_positive(List.cons(b, List.nil[Nat]))
                                all_positive_head(b, List.nil[Nat])
                                Nat.0 < b
                                lt_imp_lte_suc(Nat.0, b)
                                Nat.1 <= b
                                is_partition_non_increasing(l, Nat.3)
                                non_increasing(List.cons(a, List.cons(b, List.nil[Nat])))
                                non_increasing_second(a, b, List.nil[Nat])
                                non_increasing(List.cons(a, List.cons(b, List.nil[Nat]))) implies b <= a
                                b <= a
                                Nat.1 <= b and b <= a and a + b = Nat.3
                                two_parts_three(a, b)
                                Nat.1 <= b and b <= a and a + b = Nat.3 implies (a = Nat.2 and b = Nat.1)
                                a = Nat.2 and b = Nat.1
                                l = List.cons(a, List.cons(b, List.nil[Nat]))
                                l = List.cons(Nat.2, List.cons(Nat.1, List.nil[Nat]))
                            }
                            List.cons(c, t3) {
                                match t3 {
                                    List.nil {
                                        sum(List.cons(a, List.cons(b, List.cons(c, List.nil[Nat])))) =
                                            a + sum(List.cons(b, List.cons(c, List.nil[Nat])))
                                        sum(List.cons(b, List.cons(c, List.nil[Nat]))) =
                                            b + sum(List.cons(c, List.nil[Nat]))
                                        sum(List.cons(c, List.nil[Nat])) = c + sum(List.nil[Nat])
                                        sum(List.nil[Nat]) = Nat.0
                                        c + Nat.0 = c
                                        sum(List.cons(c, List.nil[Nat])) = c
                                        a + (b + c) = Nat.3
                                        is_partition_all_positive(l, Nat.3)
                                        all_positive(List.cons(a, List.cons(b, List.cons(c, List.nil[Nat]))))
                                        all_positive_head(a, List.cons(b, List.cons(c, List.nil[Nat])))
                                        Nat.0 < a
                                        lt_imp_lte_suc(Nat.0, a)
                                        Nat.1 <= a
                                        all_positive_tail(a, List.cons(b, List.cons(c, List.nil[Nat])))
                                        all_positive(List.cons(b, List.cons(c, List.nil[Nat])))
                                        all_positive_head(b, List.cons(c, List.nil[Nat]))
                                        Nat.0 < b
                                        lt_imp_lte_suc(Nat.0, b)
                                        Nat.1 <= b
                                        all_positive_tail(b, List.cons(c, List.nil[Nat]))
                                        all_positive(List.cons(c, List.nil[Nat]))
                                        all_positive_head(c, List.nil[Nat])
                                        Nat.0 < c
                                        lt_imp_lte_suc(Nat.0, c)
                                        Nat.1 <= c
                                        is_partition_non_increasing(l, Nat.3)
                                        non_increasing(List.cons(a, List.cons(b, List.cons(c, List.nil[Nat]))))
                                        non_increasing_second(a, b, List.cons(c, List.nil[Nat]))
                                        non_increasing(List.cons(a, List.cons(b, List.cons(c, List.nil[Nat])))) implies b <= a
                                        b <= a
                                        non_increasing_tail(a, b, List.cons(c, List.nil[Nat]))
                                        non_increasing(List.cons(a, List.cons(b, List.cons(c, List.nil[Nat])))) implies non_increasing(List.cons(b, List.cons(c, List.nil[Nat])))
                                        non_increasing(List.cons(b, List.cons(c, List.nil[Nat])))
                                        non_increasing_second(b, c, List.nil[Nat])
                                        non_increasing(List.cons(b, List.cons(c, List.nil[Nat]))) implies c <= b
                                        c <= b
                                        Nat.1 <= a and Nat.1 <= b and Nat.1 <= c and b <= a and c <= b and a + b + c = Nat.3
                                        three_parts_three(a, b, c)
                                        Nat.1 <= a and Nat.1 <= b and Nat.1 <= c and b <= a and c <= b and a + b + c = Nat.3 implies a = Nat.1 and b = Nat.1 and c = Nat.1
                                        a = Nat.1 and b = Nat.1 and c = Nat.1
                                        l = List.cons(a, List.cons(b, List.cons(c, List.nil[Nat])))
                                        l = List.cons(Nat.1, List.cons(Nat.1, List.cons(Nat.1, List.nil[Nat])))
                                    }
                                    List.cons(d, t4) {
                                        sum_ge_length_all_positive(l)
                                        all_positive(l) implies l.length <= sum(l)
                                        is_partition_all_positive(l, Nat.3)
                                        all_positive(l)
                                        l.length <= sum(l)
                                        sum(l) = Nat.3
                                        l.length <= Nat.3
                                        List.cons(a, t).length = t.length.suc
                                        l.length = t.length.suc
                                        List.cons(b, t2).length = t2.length.suc
                                        t.length = t2.length.suc
                                        List.cons(c, t3).length = t3.length.suc
                                        t2.length = t3.length.suc
                                        List.cons(d, t4).length = t4.length.suc
                                        t3.length = t4.length.suc
                                        l.length = t4.length.suc.suc.suc.suc
                                        zero_lte(t4.length)
                                        Nat.0 <= t4.length
                                        lte_suc_suc(Nat.0, t4.length)
                                        Nat.1 <= t4.length.suc
                                        lte_suc_suc(Nat.1, t4.length.suc)
                                        Nat.2 <= t4.length.suc.suc
                                        lte_suc_suc(Nat.2, t4.length.suc.suc)
                                        Nat.3 <= t4.length.suc.suc.suc
                                        lte_suc_suc(Nat.3, t4.length.suc.suc.suc)
                                        Nat.4 <= t4.length.suc.suc.suc.suc
                                        Nat.4 <= l.length
                                        lte_trans(Nat.4, l.length, Nat.3)
                                        Nat.4 <= Nat.3
                                        false
                                    }
                                }
                            }
                        }
                    }
                }
            }
        }
        l = List.cons(Nat.3, List.nil[Nat]) or
            l = List.cons(Nat.2, List.cons(Nat.1, List.nil[Nat])) or
            l = List.cons(Nat.1, List.cons(Nat.1, List.cons(Nat.1, List.nil[Nat])))
    }
}

/// The lists 3, 2 + 1 and 1 + 1 + 1 are partitions of three.
theorem partition_three_sound(l: List[Nat]) {
    (l = List.cons(Nat.3, List.nil[Nat]) or
     l = List.cons(Nat.2, List.cons(Nat.1, List.nil[Nat])) or
     l = List.cons(Nat.1, List.cons(Nat.1, List.cons(Nat.1, List.nil[Nat]))))
    implies is_partition(l, Nat.3)
} by {
    if l = List.cons(Nat.3, List.nil[Nat]) or
        l = List.cons(Nat.2, List.cons(Nat.1, List.nil[Nat])) or
        l = List.cons(Nat.1, List.cons(Nat.1, List.cons(Nat.1, List.nil[Nat]))) {
        if l = List.cons(Nat.3, List.nil[Nat]) {
            sum(List.cons(Nat.3, List.nil[Nat])) = Nat.3 + sum(List.nil[Nat])
            sum(List.nil[Nat]) = Nat.0
            Nat.3 + Nat.0 = Nat.3
            sum(List.cons(Nat.3, List.nil[Nat])) = Nat.3
            ni_single(Nat.3)
            non_increasing(List.cons(Nat.3, List.nil[Nat])) = true
            Nat.0 < Nat.3
            ap_single(Nat.3)
            all_positive(List.cons(Nat.3, List.nil[Nat])) = true
            is_partition_intro(List.cons(Nat.3, List.nil[Nat]), Nat.3)
            sum(List.cons(Nat.3, List.nil[Nat])) = Nat.3 and non_increasing(List.cons(Nat.3, List.nil[Nat])) and all_positive(List.cons(Nat.3, List.nil[Nat]))
            is_partition(List.cons(Nat.3, List.nil[Nat]), Nat.3)
            is_partition(l, Nat.3)
        }
        if l = List.cons(Nat.2, List.cons(Nat.1, List.nil[Nat])) {
            sum(List.cons(Nat.2, List.cons(Nat.1, List.nil[Nat]))) =
                Nat.2 + sum(List.cons(Nat.1, List.nil[Nat]))
            sum(List.cons(Nat.1, List.nil[Nat])) = Nat.1 + sum(List.nil[Nat])
            sum(List.nil[Nat]) = Nat.0
            Nat.1 + Nat.0 = Nat.1
            sum(List.cons(Nat.1, List.nil[Nat])) = Nat.1
            Nat.2 + Nat.1 = Nat.3
            sum(List.cons(Nat.2, List.cons(Nat.1, List.nil[Nat]))) = Nat.3
            ni_21
            non_increasing(List.cons(Nat.2, List.cons(Nat.1, List.nil[Nat]))) = true
            ap_21
            all_positive(List.cons(Nat.2, List.cons(Nat.1, List.nil[Nat]))) = true
            is_partition_intro(List.cons(Nat.2, List.cons(Nat.1, List.nil[Nat])), Nat.3)
            sum(List.cons(Nat.2, List.cons(Nat.1, List.nil[Nat]))) = Nat.3 and non_increasing(List.cons(Nat.2, List.cons(Nat.1, List.nil[Nat]))) and all_positive(List.cons(Nat.2, List.cons(Nat.1, List.nil[Nat])))
            is_partition(List.cons(Nat.2, List.cons(Nat.1, List.nil[Nat])), Nat.3)
            is_partition(l, Nat.3)
        }
        if l = List.cons(Nat.1, List.cons(Nat.1, List.cons(Nat.1, List.nil[Nat]))) {
            sum(List.cons(Nat.1, List.cons(Nat.1, List.cons(Nat.1, List.nil[Nat])))) =
                Nat.1 + sum(List.cons(Nat.1, List.cons(Nat.1, List.nil[Nat])))
            sum(List.cons(Nat.1, List.cons(Nat.1, List.nil[Nat]))) =
                Nat.1 + sum(List.cons(Nat.1, List.nil[Nat]))
            sum(List.cons(Nat.1, List.nil[Nat])) = Nat.1 + sum(List.nil[Nat])
            sum(List.nil[Nat]) = Nat.0
            Nat.1 + Nat.0 = Nat.1
            sum(List.cons(Nat.1, List.nil[Nat])) = Nat.1
            Nat.1 + Nat.1 = Nat.2
            sum(List.cons(Nat.1, List.cons(Nat.1, List.nil[Nat]))) = Nat.2
            Nat.1 + Nat.2 = Nat.3
            sum(List.cons(Nat.1, List.cons(Nat.1, List.cons(Nat.1, List.nil[Nat])))) = Nat.3
            ni_111
            non_increasing(List.cons(Nat.1, List.cons(Nat.1, List.cons(Nat.1, List.nil[Nat])))) = true
            ap_111
            all_positive(List.cons(Nat.1, List.cons(Nat.1, List.cons(Nat.1, List.nil[Nat])))) = true
            is_partition_intro(List.cons(Nat.1, List.cons(Nat.1, List.cons(Nat.1, List.nil[Nat]))), Nat.3)
            sum(List.cons(Nat.1, List.cons(Nat.1, List.cons(Nat.1, List.nil[Nat])))) = Nat.3 and non_increasing(List.cons(Nat.1, List.cons(Nat.1, List.cons(Nat.1, List.nil[Nat])))) and all_positive(List.cons(Nat.1, List.cons(Nat.1, List.cons(Nat.1, List.nil[Nat]))))
            is_partition(List.cons(Nat.1, List.cons(Nat.1, List.cons(Nat.1, List.nil[Nat]))), Nat.3)
            is_partition(l, Nat.3)
        }
        is_partition(l, Nat.3)
    }
}

/// The partitions of three are exactly 3, 2 + 1 and 1 + 1 + 1.
theorem partition_three_char(l: List[Nat]) {
    is_partition(l, Nat.3) =
        (l = List.cons(Nat.3, List.nil[Nat]) or
         l = List.cons(Nat.2, List.cons(Nat.1, List.nil[Nat])) or
         l = List.cons(Nat.1, List.cons(Nat.1, List.cons(Nat.1, List.nil[Nat]))))
} by {
    partition_three_members(l)
    is_partition(l, Nat.3) implies (l = List.cons(Nat.3, List.nil[Nat]) or l = List.cons(Nat.2, List.cons(Nat.1, List.nil[Nat])) or l = List.cons(Nat.1, List.cons(Nat.1, List.cons(Nat.1, List.nil[Nat]))))
    partition_three_sound(l)
    (l = List.cons(Nat.3, List.nil[Nat]) or l = List.cons(Nat.2, List.cons(Nat.1, List.nil[Nat])) or l = List.cons(Nat.1, List.cons(Nat.1, List.cons(Nat.1, List.nil[Nat])))) implies is_partition(l, Nat.3)
    (is_partition(l, Nat.3) implies (l = List.cons(Nat.3, List.nil[Nat]) or l = List.cons(Nat.2, List.cons(Nat.1, List.nil[Nat])) or l = List.cons(Nat.1, List.cons(Nat.1, List.cons(Nat.1, List.nil[Nat]))))) and
        ((l = List.cons(Nat.3, List.nil[Nat]) or l = List.cons(Nat.2, List.cons(Nat.1, List.nil[Nat])) or l = List.cons(Nat.1, List.cons(Nat.1, List.cons(Nat.1, List.nil[Nat])))) implies is_partition(l, Nat.3))
    bool_eq_of_iff(is_partition(l, Nat.3), l = List.cons(Nat.3, List.nil[Nat]) or l = List.cons(Nat.2, List.cons(Nat.1, List.nil[Nat])) or l = List.cons(Nat.1, List.cons(Nat.1, List.cons(Nat.1, List.nil[Nat]))))
    is_partition(l, Nat.3) = (l = List.cons(Nat.3, List.nil[Nat]) or l = List.cons(Nat.2, List.cons(Nat.1, List.nil[Nat])) or l = List.cons(Nat.1, List.cons(Nat.1, List.cons(Nat.1, List.nil[Nat]))))
}

/// The only four positive ordered parts summing to four are all ones.
theorem four_parts_four(a: Nat, b: Nat, c: Nat, d: Nat) {
    Nat.1 <= a and Nat.1 <= b and Nat.1 <= c and Nat.1 <= d and b <= a and c <= b and d <= c and
        a + b + c + d = Nat.4 implies a = Nat.1 and b = Nat.1 and c = Nat.1 and d = Nat.1
} by {
    if Nat.1 <= a and Nat.1 <= b and Nat.1 <= c and Nat.1 <= d and b <= a and c <= b and d <= c and
        a + b + c + d = Nat.4 {
        Nat.1 <= a
        Nat.1 <= b
        Nat.1 <= c
        Nat.1 <= d
        b <= a
        c <= b
        d <= c
        a + b + c + d = Nat.4
        add_assoc(a, b, c + d)
        a + (b + (c + d)) = Nat.4
        lte_add_left(c, Nat.1, d)
        Nat.1 <= d implies c + Nat.1 <= c + d
        c + Nat.1 <= c + d
        add_one_right(c)
        c + Nat.1 = c.suc
        c.suc <= c + d
        lte_add_right(Nat.1, c, Nat.1)
        Nat.1 <= c implies Nat.1 + Nat.1 <= c + Nat.1
        Nat.1 + Nat.1 = Nat.2
        Nat.2 <= c + Nat.1
        lte_trans(Nat.2, c.suc, c + d)
        Nat.2 <= c + d
        lte_add_left(b, Nat.2, c + d)
        Nat.2 <= c + d implies b + Nat.2 <= b + (c + d)
        b + Nat.2 <= b + (c + d)
        lte_add_right(Nat.2, b, Nat.1)
        Nat.1 <= b implies Nat.1 + Nat.2 <= b + Nat.2
        Nat.1 + Nat.2 = Nat.3
        Nat.3 <= b + Nat.2
        lte_trans(Nat.3, b + Nat.2, b + (c + d))
        Nat.3 <= b + (c + d)
        lte_add_left(a, Nat.3, b + (c + d))
        Nat.3 <= b + (c + d) implies a + Nat.3 <= a + (b + (c + d))
        a + Nat.3 <= a + (b + (c + d))
        a + (b + (c + d)) = Nat.4
        a + Nat.3 <= Nat.4
        add_one_right(a)
        a + Nat.1 = a.suc
        add_one_right(a.suc)
        a.suc + Nat.1 = a.suc.suc
        a + Nat.2 = a.suc.suc
        a + Nat.3 = a.suc.suc.suc
        a.suc.suc.suc <= Nat.4
        lte_cancel_suc(a.suc.suc, Nat.3)
        a.suc.suc <= Nat.3
        lte_cancel_suc(a.suc, Nat.2)
        a.suc <= Nat.2
        lte_cancel_suc(a, Nat.1)
        a <= Nat.1
        lte_antisymm(a, Nat.1)
        Nat.1 <= a and a <= Nat.1
        a = Nat.1
        a + (b + (c + d)) = Nat.4
        add_cancels_left(Nat.1, b + (c + d), Nat.3)
        b + (c + d) = Nat.3
        add_assoc(b, c, d)
        b + c + d = b + (c + d)
        b + c + d = Nat.3
        Nat.1 <= b and Nat.1 <= c and Nat.1 <= d and c <= b and d <= c and b + c + d = Nat.3
        three_parts_three(b, c, d)
        Nat.1 <= b and Nat.1 <= c and Nat.1 <= d and c <= b and d <= c and b + c + d = Nat.3 implies b = Nat.1 and c = Nat.1 and d = Nat.1
        b = Nat.1 and c = Nat.1 and d = Nat.1
        a = Nat.1 and b = Nat.1 and c = Nat.1 and d = Nat.1
    }
}

/// The partitions of four are exactly 4, 3 + 1, 2 + 2, 2 + 1 + 1 and 1 + 1 + 1 + 1.
theorem partition_four_members(l: List[Nat]) {
    is_partition(l, Nat.4) implies (l = List.cons(Nat.4, List.nil[Nat]) or
         l = List.cons(Nat.3, List.cons(Nat.1, List.nil[Nat])) or
         l = List.cons(Nat.2, List.cons(Nat.2, List.nil[Nat])) or
         l = List.cons(Nat.2, List.cons(Nat.1, List.cons(Nat.1, List.nil[Nat]))) or
         l = List.cons(Nat.1, List.cons(Nat.1, List.cons(Nat.1, List.cons(Nat.1, List.nil[Nat])))))
} by {
    if is_partition(l, Nat.4) {
        is_partition_sum(l, Nat.4)
        sum(l) = Nat.4
        match l {
            List.nil {
                sum(List.nil[Nat]) = Nat.0
                Nat.0 = Nat.4
                false
            }
            List.cons(a, t) {
                match t {
                    List.nil {
                        sum(List.cons(a, List.nil[Nat])) = a + sum(List.nil[Nat])
                        sum(List.nil[Nat]) = Nat.0
                        a + Nat.0 = a
                        sum(List.cons(a, List.nil[Nat])) = a
                        a = Nat.4
                        l = List.cons(a, List.nil[Nat])
                        l = List.cons(Nat.4, List.nil[Nat])
                    }
                    List.cons(b, t2) {
                        match t2 {
                            List.nil {
                                sum(List.cons(a, List.cons(b, List.nil[Nat]))) =
                                    a + sum(List.cons(b, List.nil[Nat]))
                                sum(List.cons(b, List.nil[Nat])) = b + sum(List.nil[Nat])
                                sum(List.nil[Nat]) = Nat.0
                                b + Nat.0 = b
                                sum(List.cons(b, List.nil[Nat])) = b
                                a + b = Nat.4
                                is_partition_all_positive(l, Nat.4)
                                all_positive(List.cons(a, List.cons(b, List.nil[Nat])))
                                all_positive_head(a, List.cons(b, List.nil[Nat]))
                                Nat.0 < a
                                lt_imp_lte_suc(Nat.0, a)
                                Nat.1 <= a
                                all_positive_tail(a, List.cons(b, List.nil[Nat]))
                                all_positive(List.cons(b, List.nil[Nat]))
                                all_positive_head(b, List.nil[Nat])
                                Nat.0 < b
                                lt_imp_lte_suc(Nat.0, b)
                                Nat.1 <= b
                                is_partition_non_increasing(l, Nat.4)
                                non_increasing(List.cons(a, List.cons(b, List.nil[Nat])))
                                non_increasing_second(a, b, List.nil[Nat])
                                non_increasing(List.cons(a, List.cons(b, List.nil[Nat]))) implies b <= a
                                b <= a
                                lt_or_lte(a, Nat.3)
                                if a < Nat.3 {
                                    Nat.1 <= b and b <= a and a + b = Nat.4 and a < Nat.3
                                    two_parts_four_small(a, b)
                                    Nat.1 <= b and b <= a and a + b = Nat.4 and a < Nat.3 implies (a = Nat.2 and b = Nat.2)
                                    a = Nat.2 and b = Nat.2
                                    l = List.cons(a, List.cons(b, List.nil[Nat]))
                                    l = List.cons(Nat.2, List.cons(Nat.2, List.nil[Nat]))
                                    or_intro_right(l = List.cons(Nat.3, List.cons(Nat.1, List.nil[Nat])), l = List.cons(Nat.2, List.cons(Nat.2, List.nil[Nat])))
                                    l = List.cons(Nat.3, List.cons(Nat.1, List.nil[Nat])) or l = List.cons(Nat.2, List.cons(Nat.2, List.nil[Nat]))
                                }
                                if not a < Nat.3 {
                                    Nat.1 <= b and b <= a and a + b = Nat.4 and not a < Nat.3
                                    two_parts_four_big(a, b)
                                    Nat.1 <= b and b <= a and a + b = Nat.4 and not a < Nat.3 implies (a = Nat.3 and b = Nat.1)
                                    a = Nat.3 and b = Nat.1
                                    l = List.cons(a, List.cons(b, List.nil[Nat]))
                                    l = List.cons(Nat.3, List.cons(Nat.1, List.nil[Nat]))
                                    or_intro_left(l = List.cons(Nat.3, List.cons(Nat.1, List.nil[Nat])), l = List.cons(Nat.2, List.cons(Nat.2, List.nil[Nat])))
                                    l = List.cons(Nat.3, List.cons(Nat.1, List.nil[Nat])) or l = List.cons(Nat.2, List.cons(Nat.2, List.nil[Nat]))
                                }
                                l = List.cons(Nat.3, List.cons(Nat.1, List.nil[Nat])) or
                                    l = List.cons(Nat.2, List.cons(Nat.2, List.nil[Nat]))
                            }
                            List.cons(c, t3) {
                                match t3 {
                                    List.nil {
                                        sum(List.cons(a, List.cons(b, List.cons(c, List.nil[Nat])))) =
                                            a + sum(List.cons(b, List.cons(c, List.nil[Nat])))
                                        sum(List.cons(b, List.cons(c, List.nil[Nat]))) =
                                            b + sum(List.cons(c, List.nil[Nat]))
                                        sum(List.cons(c, List.nil[Nat])) = c + sum(List.nil[Nat])
                                        sum(List.nil[Nat]) = Nat.0
                                        c + Nat.0 = c
                                        sum(List.cons(c, List.nil[Nat])) = c
                                        a + (b + c) = Nat.4
                                        is_partition_all_positive(l, Nat.4)
                                        all_positive(List.cons(a, List.cons(b, List.cons(c, List.nil[Nat]))))
                                        all_positive_head(a, List.cons(b, List.cons(c, List.nil[Nat])))
                                        Nat.0 < a
                                        lt_imp_lte_suc(Nat.0, a)
                                        Nat.1 <= a
                                        all_positive_tail(a, List.cons(b, List.cons(c, List.nil[Nat])))
                                        all_positive(List.cons(b, List.cons(c, List.nil[Nat])))
                                        all_positive_head(b, List.cons(c, List.nil[Nat]))
                                        Nat.0 < b
                                        lt_imp_lte_suc(Nat.0, b)
                                        Nat.1 <= b
                                        all_positive_tail(b, List.cons(c, List.nil[Nat]))
                                        all_positive(List.cons(c, List.nil[Nat]))
                                        all_positive_head(c, List.nil[Nat])
                                        Nat.0 < c
                                        lt_imp_lte_suc(Nat.0, c)
                                        Nat.1 <= c
                                        is_partition_non_increasing(l, Nat.4)
                                        non_increasing(List.cons(a, List.cons(b, List.cons(c, List.nil[Nat]))))
                                        non_increasing_second(a, b, List.cons(c, List.nil[Nat]))
                                        non_increasing(List.cons(a, List.cons(b, List.cons(c, List.nil[Nat])))) implies b <= a
                                        b <= a
                                        non_increasing_tail(a, b, List.cons(c, List.nil[Nat]))
                                        non_increasing(List.cons(a, List.cons(b, List.cons(c, List.nil[Nat])))) implies non_increasing(List.cons(b, List.cons(c, List.nil[Nat])))
                                        non_increasing(List.cons(b, List.cons(c, List.nil[Nat])))
                                        non_increasing_second(b, c, List.nil[Nat])
                                        non_increasing(List.cons(b, List.cons(c, List.nil[Nat]))) implies c <= b
                                        c <= b
                                        Nat.1 <= a and Nat.1 <= b and Nat.1 <= c and b <= a and c <= b and a + b + c = Nat.4
                                        three_parts_four(a, b, c)
                                        Nat.1 <= a and Nat.1 <= b and Nat.1 <= c and b <= a and c <= b and a + b + c = Nat.4 implies a = Nat.2 and b = Nat.1 and c = Nat.1
                                        a = Nat.2 and b = Nat.1 and c = Nat.1
                                        l = List.cons(a, List.cons(b, List.cons(c, List.nil[Nat])))
                                        l = List.cons(Nat.2, List.cons(Nat.1, List.cons(Nat.1, List.nil[Nat])))
                                    }
                                    List.cons(d, t4) {
                                        match t4 {
                                            List.nil {
                                                sum(List.cons(a, List.cons(b, List.cons(c, List.cons(d, List.nil[Nat]))))) =
                                                    a + sum(List.cons(b, List.cons(c, List.cons(d, List.nil[Nat]))))
                                                sum(List.cons(b, List.cons(c, List.cons(d, List.nil[Nat])))) =
                                                    b + sum(List.cons(c, List.cons(d, List.nil[Nat])))
                                                sum(List.cons(c, List.cons(d, List.nil[Nat]))) =
                                                    c + sum(List.cons(d, List.nil[Nat]))
                                                sum(List.cons(d, List.nil[Nat])) = d + sum(List.nil[Nat])
                                                sum(List.nil[Nat]) = Nat.0
                                                d + Nat.0 = d
                                                sum(List.cons(d, List.nil[Nat])) = d
                                                a + (b + (c + d)) = Nat.4
                                                is_partition_all_positive(l, Nat.4)
                                                all_positive(List.cons(a, List.cons(b, List.cons(c, List.cons(d, List.nil[Nat])))))
                                                all_positive_head(a, List.cons(b, List.cons(c, List.cons(d, List.nil[Nat]))))
                                                Nat.0 < a
                                                lt_imp_lte_suc(Nat.0, a)
                                                Nat.1 <= a
                                                all_positive_tail(a, List.cons(b, List.cons(c, List.cons(d, List.nil[Nat]))))
                                                all_positive(List.cons(b, List.cons(c, List.cons(d, List.nil[Nat]))))
                                                all_positive_head(b, List.cons(c, List.cons(d, List.nil[Nat])))
                                                Nat.0 < b
                                                lt_imp_lte_suc(Nat.0, b)
                                                Nat.1 <= b
                                                all_positive_tail(b, List.cons(c, List.cons(d, List.nil[Nat])))
                                                all_positive(List.cons(c, List.cons(d, List.nil[Nat])))
                                                all_positive_head(c, List.cons(d, List.nil[Nat]))
                                                Nat.0 < c
                                                lt_imp_lte_suc(Nat.0, c)
                                                Nat.1 <= c
                                                all_positive_tail(c, List.cons(d, List.nil[Nat]))
                                                all_positive(List.cons(d, List.nil[Nat]))
                                                all_positive_head(d, List.nil[Nat])
                                                Nat.0 < d
                                                lt_imp_lte_suc(Nat.0, d)
                                                Nat.1 <= d
                                                is_partition_non_increasing(l, Nat.4)
                                                non_increasing(List.cons(a, List.cons(b, List.cons(c, List.cons(d, List.nil[Nat])))))
                                                non_increasing_second(a, b, List.cons(c, List.cons(d, List.nil[Nat])))
                                                non_increasing(List.cons(a, List.cons(b, List.cons(c, List.cons(d, List.nil[Nat]))))) implies b <= a
                                                b <= a
                                                non_increasing_tail(a, b, List.cons(c, List.cons(d, List.nil[Nat])))
                                                non_increasing(List.cons(a, List.cons(b, List.cons(c, List.cons(d, List.nil[Nat]))))) implies non_increasing(List.cons(b, List.cons(c, List.cons(d, List.nil[Nat]))))
                                                non_increasing(List.cons(b, List.cons(c, List.cons(d, List.nil[Nat]))))
                                                non_increasing_second(b, c, List.cons(d, List.nil[Nat]))
                                                non_increasing(List.cons(b, List.cons(c, List.cons(d, List.nil[Nat])))) implies c <= b
                                                c <= b
                                                non_increasing_tail(b, c, List.cons(d, List.nil[Nat]))
                                                non_increasing(List.cons(b, List.cons(c, List.cons(d, List.nil[Nat])))) implies non_increasing(List.cons(c, List.cons(d, List.nil[Nat])))
                                                non_increasing(List.cons(c, List.cons(d, List.nil[Nat])))
                                                non_increasing_second(c, d, List.nil[Nat])
                                                non_increasing(List.cons(c, List.cons(d, List.nil[Nat]))) implies d <= c
                                                d <= c
                                                Nat.1 <= a and Nat.1 <= b and Nat.1 <= c and Nat.1 <= d and b <= a and c <= b and d <= c and a + b + c + d = Nat.4
                                                four_parts_four(a, b, c, d)
                                                Nat.1 <= a and Nat.1 <= b and Nat.1 <= c and Nat.1 <= d and b <= a and c <= b and d <= c and a + b + c + d = Nat.4 implies a = Nat.1 and b = Nat.1 and c = Nat.1 and d = Nat.1
                                                a = Nat.1 and b = Nat.1 and c = Nat.1 and d = Nat.1
                                                l = List.cons(a, List.cons(b, List.cons(c, List.cons(d, List.nil[Nat]))))
                                                l = List.cons(Nat.1, List.cons(Nat.1, List.cons(Nat.1, List.cons(Nat.1, List.nil[Nat]))))
                                            }
                                            List.cons(e, t5) {
                                                sum_ge_length_all_positive(l)
                                                all_positive(l) implies l.length <= sum(l)
                                                is_partition_all_positive(l, Nat.4)
                                                all_positive(l)
                                                l.length <= sum(l)
                                                sum(l) = Nat.4
                                                l.length <= Nat.4
                                                List.cons(a, t).length = t.length.suc
                                                l.length = t.length.suc
                                                List.cons(b, t2).length = t2.length.suc
                                                t.length = t2.length.suc
                                                List.cons(c, t3).length = t3.length.suc
                                                t2.length = t3.length.suc
                                                List.cons(d, t4).length = t4.length.suc
                                                t3.length = t4.length.suc
                                                List.cons(e, t5).length = t5.length.suc
                                                t4.length = t5.length.suc
                                                l.length = t5.length.suc.suc.suc.suc.suc
                                                zero_lte(t5.length)
                                                Nat.0 <= t5.length
                                                lte_suc_suc(Nat.0, t5.length)
                                                Nat.1 <= t5.length.suc
                                                lte_suc_suc(Nat.1, t5.length.suc)
                                                Nat.2 <= t5.length.suc.suc
                                                lte_suc_suc(Nat.2, t5.length.suc.suc)
                                                Nat.3 <= t5.length.suc.suc.suc
                                                lte_suc_suc(Nat.3, t5.length.suc.suc.suc)
                                                Nat.4 <= t5.length.suc.suc.suc.suc
                                                lte_suc_suc(Nat.4, t5.length.suc.suc.suc.suc)
                                                Nat.5 <= t5.length.suc.suc.suc.suc.suc
                                                Nat.5 <= l.length
                                                lte_trans(Nat.5, l.length, Nat.4)
                                                Nat.5 <= Nat.4
                                                lte_imp_not_lt(Nat.5, Nat.4)
                                                Nat.5 <= Nat.4 implies not Nat.4 < Nat.5
                                                not Nat.4 < Nat.5
                                                lt_suc(Nat.4)
                                                Nat.4 < Nat.5
                                                false
                                            }
                                        }
                                    }
                                }
                            }
                        }
                    }
                }
            }
        }
        l = List.cons(Nat.4, List.nil[Nat]) or
            l = List.cons(Nat.3, List.cons(Nat.1, List.nil[Nat])) or
            l = List.cons(Nat.2, List.cons(Nat.2, List.nil[Nat])) or
            l = List.cons(Nat.2, List.cons(Nat.1, List.cons(Nat.1, List.nil[Nat]))) or
            l = List.cons(Nat.1, List.cons(Nat.1, List.cons(Nat.1, List.cons(Nat.1, List.nil[Nat]))))
    }
}

/// The lists 4, 3 + 1, 2 + 2, 2 + 1 + 1 and 1 + 1 + 1 + 1 are partitions of four.
theorem partition_four_sound(l: List[Nat]) {
    (l = List.cons(Nat.4, List.nil[Nat]) or
     l = List.cons(Nat.3, List.cons(Nat.1, List.nil[Nat])) or
     l = List.cons(Nat.2, List.cons(Nat.2, List.nil[Nat])) or
     l = List.cons(Nat.2, List.cons(Nat.1, List.cons(Nat.1, List.nil[Nat]))) or
     l = List.cons(Nat.1, List.cons(Nat.1, List.cons(Nat.1, List.cons(Nat.1, List.nil[Nat])))))
    implies is_partition(l, Nat.4)
} by {
    if l = List.cons(Nat.4, List.nil[Nat]) or
        l = List.cons(Nat.3, List.cons(Nat.1, List.nil[Nat])) or
        l = List.cons(Nat.2, List.cons(Nat.2, List.nil[Nat])) or
        l = List.cons(Nat.2, List.cons(Nat.1, List.cons(Nat.1, List.nil[Nat]))) or
        l = List.cons(Nat.1, List.cons(Nat.1, List.cons(Nat.1, List.cons(Nat.1, List.nil[Nat])))) {
        if l = List.cons(Nat.4, List.nil[Nat]) {
            sum(List.cons(Nat.4, List.nil[Nat])) = Nat.4 + sum(List.nil[Nat])
            sum(List.nil[Nat]) = Nat.0
            Nat.4 + Nat.0 = Nat.4
            sum(List.cons(Nat.4, List.nil[Nat])) = Nat.4
            ni_single(Nat.4)
            non_increasing(List.cons(Nat.4, List.nil[Nat])) = true
            Nat.0 < Nat.4
            ap_single(Nat.4)
            all_positive(List.cons(Nat.4, List.nil[Nat])) = true
            is_partition_intro(List.cons(Nat.4, List.nil[Nat]), Nat.4)
            sum(List.cons(Nat.4, List.nil[Nat])) = Nat.4 and non_increasing(List.cons(Nat.4, List.nil[Nat])) and all_positive(List.cons(Nat.4, List.nil[Nat]))
            is_partition(List.cons(Nat.4, List.nil[Nat]), Nat.4)
            is_partition(l, Nat.4)
        }
        if l = List.cons(Nat.3, List.cons(Nat.1, List.nil[Nat])) {
            sum(List.cons(Nat.3, List.cons(Nat.1, List.nil[Nat]))) =
                Nat.3 + sum(List.cons(Nat.1, List.nil[Nat]))
            sum(List.cons(Nat.1, List.nil[Nat])) = Nat.1 + sum(List.nil[Nat])
            sum(List.nil[Nat]) = Nat.0
            Nat.1 + Nat.0 = Nat.1
            sum(List.cons(Nat.1, List.nil[Nat])) = Nat.1
            Nat.3 + Nat.1 = Nat.4
            sum(List.cons(Nat.3, List.cons(Nat.1, List.nil[Nat]))) = Nat.4
            ni_31
            non_increasing(List.cons(Nat.3, List.cons(Nat.1, List.nil[Nat]))) = true
            ap_31
            all_positive(List.cons(Nat.3, List.cons(Nat.1, List.nil[Nat]))) = true
            is_partition_intro(List.cons(Nat.3, List.cons(Nat.1, List.nil[Nat])), Nat.4)
            sum(List.cons(Nat.3, List.cons(Nat.1, List.nil[Nat]))) = Nat.4 and non_increasing(List.cons(Nat.3, List.cons(Nat.1, List.nil[Nat]))) and all_positive(List.cons(Nat.3, List.cons(Nat.1, List.nil[Nat])))
            is_partition(List.cons(Nat.3, List.cons(Nat.1, List.nil[Nat])), Nat.4)
            is_partition(l, Nat.4)
        }
        if l = List.cons(Nat.2, List.cons(Nat.2, List.nil[Nat])) {
            sum(List.cons(Nat.2, List.cons(Nat.2, List.nil[Nat]))) =
                Nat.2 + sum(List.cons(Nat.2, List.nil[Nat]))
            sum(List.cons(Nat.2, List.nil[Nat])) = Nat.2 + sum(List.nil[Nat])
            sum(List.nil[Nat]) = Nat.0
            Nat.2 + Nat.0 = Nat.2
            sum(List.cons(Nat.2, List.nil[Nat])) = Nat.2
            Nat.2 + Nat.2 = Nat.4
            sum(List.cons(Nat.2, List.cons(Nat.2, List.nil[Nat]))) = Nat.4
            ni_22
            non_increasing(List.cons(Nat.2, List.cons(Nat.2, List.nil[Nat]))) = true
            ap_22
            all_positive(List.cons(Nat.2, List.cons(Nat.2, List.nil[Nat]))) = true
            is_partition_intro(List.cons(Nat.2, List.cons(Nat.2, List.nil[Nat])), Nat.4)
            sum(List.cons(Nat.2, List.cons(Nat.2, List.nil[Nat]))) = Nat.4 and non_increasing(List.cons(Nat.2, List.cons(Nat.2, List.nil[Nat]))) and all_positive(List.cons(Nat.2, List.cons(Nat.2, List.nil[Nat])))
            is_partition(List.cons(Nat.2, List.cons(Nat.2, List.nil[Nat])), Nat.4)
            is_partition(l, Nat.4)
        }
        if l = List.cons(Nat.2, List.cons(Nat.1, List.cons(Nat.1, List.nil[Nat]))) {
            sum(List.cons(Nat.2, List.cons(Nat.1, List.cons(Nat.1, List.nil[Nat])))) =
                Nat.2 + sum(List.cons(Nat.1, List.cons(Nat.1, List.nil[Nat])))
            sum(List.cons(Nat.1, List.cons(Nat.1, List.nil[Nat]))) =
                Nat.1 + sum(List.cons(Nat.1, List.nil[Nat]))
            sum(List.cons(Nat.1, List.nil[Nat])) = Nat.1 + sum(List.nil[Nat])
            sum(List.nil[Nat]) = Nat.0
            Nat.1 + Nat.0 = Nat.1
            sum(List.cons(Nat.1, List.nil[Nat])) = Nat.1
            Nat.1 + Nat.1 = Nat.2
            sum(List.cons(Nat.1, List.cons(Nat.1, List.nil[Nat]))) = Nat.2
            Nat.2 + Nat.2 = Nat.4
            sum(List.cons(Nat.2, List.cons(Nat.1, List.cons(Nat.1, List.nil[Nat])))) = Nat.4
            ni_211
            non_increasing(List.cons(Nat.2, List.cons(Nat.1, List.cons(Nat.1, List.nil[Nat])))) = true
            ap_211
            all_positive(List.cons(Nat.2, List.cons(Nat.1, List.cons(Nat.1, List.nil[Nat])))) = true
            is_partition_intro(List.cons(Nat.2, List.cons(Nat.1, List.cons(Nat.1, List.nil[Nat]))), Nat.4)
            sum(List.cons(Nat.2, List.cons(Nat.1, List.cons(Nat.1, List.nil[Nat])))) = Nat.4 and non_increasing(List.cons(Nat.2, List.cons(Nat.1, List.cons(Nat.1, List.nil[Nat])))) and all_positive(List.cons(Nat.2, List.cons(Nat.1, List.cons(Nat.1, List.nil[Nat]))))
            is_partition(List.cons(Nat.2, List.cons(Nat.1, List.cons(Nat.1, List.nil[Nat]))), Nat.4)
            is_partition(l, Nat.4)
        }
        if l = List.cons(Nat.1, List.cons(Nat.1, List.cons(Nat.1, List.cons(Nat.1, List.nil[Nat])))) {
            sum(List.cons(Nat.1, List.cons(Nat.1, List.cons(Nat.1, List.cons(Nat.1, List.nil[Nat]))))) =
                Nat.1 + sum(List.cons(Nat.1, List.cons(Nat.1, List.cons(Nat.1, List.nil[Nat]))))
            sum(List.cons(Nat.1, List.cons(Nat.1, List.cons(Nat.1, List.nil[Nat])))) =
                Nat.1 + sum(List.cons(Nat.1, List.cons(Nat.1, List.nil[Nat])))
            sum(List.cons(Nat.1, List.cons(Nat.1, List.nil[Nat]))) =
                Nat.1 + sum(List.cons(Nat.1, List.nil[Nat]))
            sum(List.cons(Nat.1, List.nil[Nat])) = Nat.1 + sum(List.nil[Nat])
            sum(List.nil[Nat]) = Nat.0
            Nat.1 + Nat.0 = Nat.1
            sum(List.cons(Nat.1, List.nil[Nat])) = Nat.1
            Nat.1 + Nat.1 = Nat.2
            sum(List.cons(Nat.1, List.cons(Nat.1, List.nil[Nat]))) = Nat.2
            Nat.1 + Nat.2 = Nat.3
            sum(List.cons(Nat.1, List.cons(Nat.1, List.cons(Nat.1, List.nil[Nat])))) = Nat.3
            Nat.1 + Nat.3 = Nat.4
            sum(List.cons(Nat.1, List.cons(Nat.1, List.cons(Nat.1, List.cons(Nat.1, List.nil[Nat]))))) = Nat.4
            ni_1111
            non_increasing(List.cons(Nat.1, List.cons(Nat.1, List.cons(Nat.1, List.cons(Nat.1, List.nil[Nat]))))) = true
            ap_1111
            all_positive(List.cons(Nat.1, List.cons(Nat.1, List.cons(Nat.1, List.cons(Nat.1, List.nil[Nat]))))) = true
            is_partition_intro(List.cons(Nat.1, List.cons(Nat.1, List.cons(Nat.1, List.cons(Nat.1, List.nil[Nat])))), Nat.4)
            sum(List.cons(Nat.1, List.cons(Nat.1, List.cons(Nat.1, List.cons(Nat.1, List.nil[Nat]))))) = Nat.4 and non_increasing(List.cons(Nat.1, List.cons(Nat.1, List.cons(Nat.1, List.cons(Nat.1, List.nil[Nat]))))) and all_positive(List.cons(Nat.1, List.cons(Nat.1, List.cons(Nat.1, List.cons(Nat.1, List.nil[Nat])))))
            is_partition(List.cons(Nat.1, List.cons(Nat.1, List.cons(Nat.1, List.cons(Nat.1, List.nil[Nat])))), Nat.4)
            is_partition(l, Nat.4)
        }
        is_partition(l, Nat.4)
    }
}

/// The partitions of four are exactly 4, 3 + 1, 2 + 2, 2 + 1 + 1 and 1 + 1 + 1 + 1.
theorem partition_four_char(l: List[Nat]) {
    is_partition(l, Nat.4) =
        (l = List.cons(Nat.4, List.nil[Nat]) or
         l = List.cons(Nat.3, List.cons(Nat.1, List.nil[Nat])) or
         l = List.cons(Nat.2, List.cons(Nat.2, List.nil[Nat])) or
         l = List.cons(Nat.2, List.cons(Nat.1, List.cons(Nat.1, List.nil[Nat]))) or
         l = List.cons(Nat.1, List.cons(Nat.1, List.cons(Nat.1, List.cons(Nat.1, List.nil[Nat])))))
} by {
    partition_four_members(l)
    is_partition(l, Nat.4) implies (l = List.cons(Nat.4, List.nil[Nat]) or l = List.cons(Nat.3, List.cons(Nat.1, List.nil[Nat])) or l = List.cons(Nat.2, List.cons(Nat.2, List.nil[Nat])) or l = List.cons(Nat.2, List.cons(Nat.1, List.cons(Nat.1, List.nil[Nat]))) or l = List.cons(Nat.1, List.cons(Nat.1, List.cons(Nat.1, List.cons(Nat.1, List.nil[Nat])))))
    partition_four_sound(l)
    (l = List.cons(Nat.4, List.nil[Nat]) or l = List.cons(Nat.3, List.cons(Nat.1, List.nil[Nat])) or l = List.cons(Nat.2, List.cons(Nat.2, List.nil[Nat])) or l = List.cons(Nat.2, List.cons(Nat.1, List.cons(Nat.1, List.nil[Nat]))) or l = List.cons(Nat.1, List.cons(Nat.1, List.cons(Nat.1, List.cons(Nat.1, List.nil[Nat]))))) implies is_partition(l, Nat.4)
    (is_partition(l, Nat.4) implies (l = List.cons(Nat.4, List.nil[Nat]) or l = List.cons(Nat.3, List.cons(Nat.1, List.nil[Nat])) or l = List.cons(Nat.2, List.cons(Nat.2, List.nil[Nat])) or l = List.cons(Nat.2, List.cons(Nat.1, List.cons(Nat.1, List.nil[Nat]))) or l = List.cons(Nat.1, List.cons(Nat.1, List.cons(Nat.1, List.cons(Nat.1, List.nil[Nat])))))) and
        ((l = List.cons(Nat.4, List.nil[Nat]) or l = List.cons(Nat.3, List.cons(Nat.1, List.nil[Nat])) or l = List.cons(Nat.2, List.cons(Nat.2, List.nil[Nat])) or l = List.cons(Nat.2, List.cons(Nat.1, List.cons(Nat.1, List.nil[Nat]))) or l = List.cons(Nat.1, List.cons(Nat.1, List.cons(Nat.1, List.cons(Nat.1, List.nil[Nat]))))) implies is_partition(l, Nat.4))
    bool_eq_of_iff(is_partition(l, Nat.4), l = List.cons(Nat.4, List.nil[Nat]) or l = List.cons(Nat.3, List.cons(Nat.1, List.nil[Nat])) or l = List.cons(Nat.2, List.cons(Nat.2, List.nil[Nat])) or l = List.cons(Nat.2, List.cons(Nat.1, List.cons(Nat.1, List.nil[Nat]))) or l = List.cons(Nat.1, List.cons(Nat.1, List.cons(Nat.1, List.cons(Nat.1, List.nil[Nat])))))
    is_partition(l, Nat.4) = (l = List.cons(Nat.4, List.nil[Nat]) or l = List.cons(Nat.3, List.cons(Nat.1, List.nil[Nat])) or l = List.cons(Nat.2, List.cons(Nat.2, List.nil[Nat])) or l = List.cons(Nat.2, List.cons(Nat.1, List.cons(Nat.1, List.nil[Nat]))) or l = List.cons(Nat.1, List.cons(Nat.1, List.cons(Nat.1, List.cons(Nat.1, List.nil[Nat])))))
}

// ---------------------------------------------------------------------------
// The number of partitions, as the cardinality of the set of partitions.
// ---------------------------------------------------------------------------

/// True if `parts` is a partition of `n`; the membership predicate of the
/// set of partitions of `n`.
define is_partition_of_n(n: Nat, parts: List[Nat]) -> Bool {
    is_partition(parts, n)
}

/// The set of all partitions of `n`.
define partition_set(n: Nat) -> Set[List[Nat]] {
    Set.new(is_partition_of_n(n))
}

/// Membership in the set of partitions of `n` is the partition predicate.
theorem partition_set_contains_eq(n: Nat, parts: List[Nat]) {
    partition_set(n).contains(parts) = is_partition_of_n(n, parts)
}

/// Membership in the set of partitions of `n` means being a partition of `n`.
theorem partition_set_contains_iff(n: Nat, parts: List[Nat]) {
    partition_set(n).contains(parts) = is_partition(parts, n)
} by {
    partition_set_contains_eq(n, parts)
    partition_set(n).contains(parts) = is_partition_of_n(n, parts)
    is_partition_of_n(n, parts) = is_partition(parts, n)
    partition_set(n).contains(parts) = is_partition(parts, n)
}

/// The lists obtained by prepending `a` to each inner list.
define prepend_a(a: Nat, inners: List[List[Nat]]) -> List[List[Nat]] {
    map(inners, function(rest: List[Nat]) { List.cons(a, rest) })
}

/// The lists obtained by prepending an alphabet element to each inner list.
define prepend_alphabet(alphabet: List[Nat], inners: List[List[Nat]]) -> List[List[Nat]] {
    match alphabet {
        List.nil {
            List.nil[List[Nat]]
        }
        List.cons(a, t) {
            prepend_a(a, inners) + prepend_alphabet(t, inners)
        }
    }
}

/// All lists of exactly `len` elements drawn from `alphabet`.
define lists_of_length(alphabet: List[Nat], len: Nat) -> List[List[Nat]] {
    match len {
        Nat.zero {
            List.singleton[List[Nat]](List.nil[Nat])
        }
        Nat.suc(k) {
            prepend_alphabet(alphabet, lists_of_length(alphabet, k))
        }
    }
}

/// All lists of length at most `max_len` drawn from `alphabet`.
define all_lengths(alphabet: List[Nat], max_len: Nat) -> List[List[Nat]] {
    match max_len {
        Nat.zero {
            lists_of_length(alphabet, Nat.zero)
        }
        Nat.suc(k) {
            all_lengths(alphabet, k) + lists_of_length(alphabet, k.suc)
        }
    }
}

/// Prepending an alphabet element to an inner list keeps the resulting list
/// in the prepended family.
theorem prepend_alphabet_contains(alphabet: List[Nat], inners: List[List[Nat]], a: Nat, rest: List[Nat]) {
    alphabet.contains(a) and inners.contains(rest) implies prepend_alphabet(alphabet, inners).contains(List.cons(a, rest))
} by {
    define q(xs: List[Nat], k: Nat, ins: List[List[Nat]], r: List[Nat]) -> Bool {
        xs.contains(k) and ins.contains(r) implies prepend_alphabet(xs, ins).contains(List.cons(k, r))
    }
    if List.nil[Nat].contains(a) and inners.contains(rest) {
        false
    }
    q(List.nil[Nat], a, inners, rest)
    forall(h: Nat, t: List[Nat]) {
        if q(t, a, inners, rest) {
            if List.cons(h, t).contains(a) and inners.contains(rest) {
                if h = a {
                    prepend_alphabet(List.cons(h, t), inners) =
                        prepend_a(h, inners) + prepend_alphabet(t, inners)
                    inners.contains(rest)
                    map_contains_of_contains(inners, function(r: List[Nat]) { List.cons(h, r) }, rest)
                    inners.contains(rest) implies map(inners, function(r: List[Nat]) { List.cons(h, r) }).contains(List.cons(h, rest))
                    map(inners, function(r: List[Nat]) { List.cons(h, r) }).contains(List.cons(h, rest))
                    prepend_a(h, inners) = map(inners, function(r: List[Nat]) { List.cons(h, r) })
                    prepend_a(h, inners).contains(List.cons(h, rest))
                    List.cons(h, rest) = List.cons(a, rest)
                    prepend_a(h, inners).contains(List.cons(a, rest))
                    add_contains_left(prepend_a(h, inners), prepend_alphabet(t, inners), List.cons(a, rest))
                    prepend_a(h, inners).contains(List.cons(a, rest)) implies (prepend_a(h, inners) + prepend_alphabet(t, inners)).contains(List.cons(a, rest))
                    (prepend_a(h, inners) + prepend_alphabet(t, inners)).contains(List.cons(a, rest))
                    prepend_alphabet(List.cons(h, t), inners).contains(List.cons(a, rest))
                }
                if not h = a {
                    List.cons(h, t).contains(a) implies t.contains(a)
                    t.contains(a)
                    q(t, a, inners, rest)
                    t.contains(a) and inners.contains(rest) implies prepend_alphabet(t, inners).contains(List.cons(a, rest))
                    t.contains(a) and inners.contains(rest)
                    prepend_alphabet(t, inners).contains(List.cons(a, rest))
                    add_contains_right(prepend_a(h, inners), prepend_alphabet(t, inners), List.cons(a, rest))
                    prepend_alphabet(t, inners).contains(List.cons(a, rest)) implies (prepend_a(h, inners) + prepend_alphabet(t, inners)).contains(List.cons(a, rest))
                    (prepend_a(h, inners) + prepend_alphabet(t, inners)).contains(List.cons(a, rest))
                    prepend_alphabet(List.cons(h, t), inners).contains(List.cons(a, rest))
                }
                prepend_alphabet(List.cons(h, t), inners).contains(List.cons(a, rest))
            }
            q(List.cons(h, t), a, inners, rest)
        }
    }
    List.induction(function(xs: List[Nat]) { q(xs, a, inners, rest) })
    q(List.nil[Nat], a, inners, rest) and forall(h: Nat, t: List[Nat]) { q(t, a, inners, rest) implies q(List.cons(h, t), a, inners, rest) } implies forall(xs: List[Nat]) { q(xs, a, inners, rest) }
    q(List.nil[Nat], a, inners, rest) and forall(h: Nat, t: List[Nat]) { q(t, a, inners, rest) implies q(List.cons(h, t), a, inners, rest) }
    forall(xs: List[Nat]) { q(xs, a, inners, rest) }
    q(alphabet, a, inners, rest)
    if alphabet.contains(a) and inners.contains(rest) {
        prepend_alphabet(alphabet, inners).contains(List.cons(a, rest))
    }
}

theorem lists_of_length_contains(alphabet: List[Nat], parts: List[Nat], len: Nat) {
    parts.length = len and (forall(x: Nat) { parts.contains(x) implies alphabet.contains(x) })
    implies lists_of_length(alphabet, len).contains(parts)
} by {
    define p(xs: List[Nat]) -> Bool {
        (forall(x: Nat) { xs.contains(x) implies alphabet.contains(x) })
        implies lists_of_length(alphabet, xs.length).contains(xs)
    }
    if forall(x: Nat) { List.nil[Nat].contains(x) implies alphabet.contains(x) } {
        lists_of_length(alphabet, Nat.zero) = List.singleton[List[Nat]](List.nil[Nat])
        List.singleton[List[Nat]](List.nil[Nat]).contains(List.nil[Nat])
        lists_of_length(alphabet, Nat.zero).contains(List.nil[Nat])
        List.nil[Nat].length = Nat.zero
        lists_of_length(alphabet, List.nil[Nat].length).contains(List.nil[Nat])
    }
    p(List.nil[Nat])
    forall(head: Nat, tail: List[Nat]) {
        if p(tail) {
            if forall(x: Nat) { List.cons(head, tail).contains(x) implies alphabet.contains(x) } {
                forall(x: Nat) {
                    if tail.contains(x) {
                        List.cons(head, tail).contains(x)
                        alphabet.contains(x)
                    }
                }
                if forall(x: Nat) { tail.contains(x) implies alphabet.contains(x) } {
                    p(tail)
                    (forall(x: Nat) { tail.contains(x) implies alphabet.contains(x) }) implies lists_of_length(alphabet, tail.length).contains(tail)
                    forall(x: Nat) { tail.contains(x) implies alphabet.contains(x) }
                    lists_of_length(alphabet, tail.length).contains(tail)
                }
                List.cons(head, tail).contains(head)
                alphabet.contains(head)
                List.cons(head, tail).length = tail.length.suc
                lists_of_length(alphabet, tail.length.suc) =
                    prepend_alphabet(alphabet, lists_of_length(alphabet, tail.length))
                alphabet.contains(head) and lists_of_length(alphabet, tail.length).contains(tail)
                prepend_alphabet_contains(alphabet, lists_of_length(alphabet, tail.length), head, tail)
                alphabet.contains(head) and lists_of_length(alphabet, tail.length).contains(tail) implies prepend_alphabet(alphabet, lists_of_length(alphabet, tail.length)).contains(List.cons(head, tail))
                prepend_alphabet(alphabet, lists_of_length(alphabet, tail.length)).contains(List.cons(head, tail))
                lists_of_length(alphabet, List.cons(head, tail).length).contains(List.cons(head, tail))
            }
            p(List.cons(head, tail))
        }
    }
    List.induction(function(xs: List[Nat]) { p(xs) })
    p(List.nil[Nat]) and forall(h: Nat, t: List[Nat]) { p(t) implies p(List.cons(h, t)) } implies forall(xs: List[Nat]) { p(xs) }
    p(List.nil[Nat]) and forall(h: Nat, t: List[Nat]) { p(t) implies p(List.cons(h, t)) }
    forall(xs: List[Nat]) { p(xs) }
    p(parts)
    if parts.length = len and (forall(x: Nat) { parts.contains(x) implies alphabet.contains(x) }) {
        forall(x: Nat) { parts.contains(x) implies alphabet.contains(x) }
        p(parts)
        (forall(x: Nat) { parts.contains(x) implies alphabet.contains(x) }) implies lists_of_length(alphabet, parts.length).contains(parts)
        lists_of_length(alphabet, parts.length).contains(parts)
        lists_of_length(alphabet, len).contains(parts)
    }
}

/// A list of length at most `max_len` over the alphabet occurs in the
/// bounded-length family.
theorem all_lengths_contains(alphabet: List[Nat], parts: List[Nat], max_len: Nat) {
    parts.length <= max_len and (forall(x: Nat) { parts.contains(x) implies alphabet.contains(x) })
    implies all_lengths(alphabet, max_len).contains(parts)
} by {
    define p(k: Nat) -> Bool {
        parts.length <= k and (forall(x: Nat) { parts.contains(x) implies alphabet.contains(x) })
        implies all_lengths(alphabet, k).contains(parts)
    }
    if parts.length <= Nat.zero and (forall(x: Nat) { parts.contains(x) implies alphabet.contains(x) }) {
        parts.length = Nat.zero
        forall(x: Nat) { parts.contains(x) implies alphabet.contains(x) }
        lists_of_length_contains(alphabet, parts, Nat.zero)
        parts.length = Nat.zero and (forall(x: Nat) { parts.contains(x) implies alphabet.contains(x) })
        lists_of_length(alphabet, Nat.zero).contains(parts)
        all_lengths(alphabet, Nat.zero) = lists_of_length(alphabet, Nat.zero)
        all_lengths(alphabet, Nat.zero).contains(parts)
    }
    p(Nat.zero)
    forall(k: Nat) {
        if p(k) {
            if parts.length <= k.suc and (forall(x: Nat) { parts.contains(x) implies alphabet.contains(x) }) {
                lt_or_lte(parts.length, k.suc)
                parts.length < k.suc or k.suc <= parts.length
                if parts.length < k.suc {
                    lt_suc_right(parts.length, k)
                    parts.length < k.suc implies parts.length = k or parts.length < k
                    parts.length = k or parts.length < k
                    if parts.length = k {
                        parts.length <= k
                        p(k)
                        parts.length <= k and (forall(x: Nat) { parts.contains(x) implies alphabet.contains(x) }) implies all_lengths(alphabet, k).contains(parts)
                        forall(x: Nat) { parts.contains(x) implies alphabet.contains(x) }
                        parts.length <= k and (forall(x: Nat) { parts.contains(x) implies alphabet.contains(x) })
                        all_lengths(alphabet, k).contains(parts)
                        add_contains_left(all_lengths(alphabet, k), lists_of_length(alphabet, k.suc), parts)
                        all_lengths(alphabet, k).contains(parts) implies (all_lengths(alphabet, k) + lists_of_length(alphabet, k.suc)).contains(parts)
                        (all_lengths(alphabet, k) + lists_of_length(alphabet, k.suc)).contains(parts)
                        all_lengths(alphabet, k.suc) = all_lengths(alphabet, k) + lists_of_length(alphabet, k.suc)
                        all_lengths(alphabet, k.suc).contains(parts)
                    }
                    if not parts.length = k {
                        parts.length < k
                        lt_imp_lte_suc(parts.length, k)
                        parts.length <= k
                        p(k)
                        parts.length <= k and (forall(x: Nat) { parts.contains(x) implies alphabet.contains(x) }) implies all_lengths(alphabet, k).contains(parts)
                        forall(x: Nat) { parts.contains(x) implies alphabet.contains(x) }
                        parts.length <= k and (forall(x: Nat) { parts.contains(x) implies alphabet.contains(x) })
                        all_lengths(alphabet, k).contains(parts)
                        add_contains_left(all_lengths(alphabet, k), lists_of_length(alphabet, k.suc), parts)
                        all_lengths(alphabet, k).contains(parts) implies (all_lengths(alphabet, k) + lists_of_length(alphabet, k.suc)).contains(parts)
                        (all_lengths(alphabet, k) + lists_of_length(alphabet, k.suc)).contains(parts)
                        all_lengths(alphabet, k.suc) = all_lengths(alphabet, k) + lists_of_length(alphabet, k.suc)
                        all_lengths(alphabet, k.suc).contains(parts)
                    }
                    all_lengths(alphabet, k.suc).contains(parts)
                }
                if not parts.length < k.suc {
                    lte_antisymm(parts.length, k.suc)
                    parts.length = k.suc
                    forall(x: Nat) { parts.contains(x) implies alphabet.contains(x) }
                    lists_of_length_contains(alphabet, parts, k.suc)
                    parts.length = k.suc and (forall(x: Nat) { parts.contains(x) implies alphabet.contains(x) }) implies lists_of_length(alphabet, k.suc).contains(parts)
                    lists_of_length(alphabet, k.suc).contains(parts)
                    add_contains_right(all_lengths(alphabet, k), lists_of_length(alphabet, k.suc), parts)
                    lists_of_length(alphabet, k.suc).contains(parts) implies (all_lengths(alphabet, k) + lists_of_length(alphabet, k.suc)).contains(parts)
                    (all_lengths(alphabet, k) + lists_of_length(alphabet, k.suc)).contains(parts)
                    all_lengths(alphabet, k.suc) = all_lengths(alphabet, k) + lists_of_length(alphabet, k.suc)
                    all_lengths(alphabet, k.suc).contains(parts)
                }
                all_lengths(alphabet, k.suc).contains(parts)
            }
            p(k.suc)
        }
    }
    p(Nat.zero) and forall(k: Nat) { p(k) implies p(k.suc) }
    Nat.induction(p)
    p(max_len)
}

/// Every element of a list is at most its sum.
theorem contains_le_sum(l: List[Nat], x: Nat) {
    l.contains(x) implies x <= sum(l)
} by {
    define r(xs: List[Nat]) -> Bool {
        xs.contains(x) implies x <= sum(xs)
    }
    if List.nil[Nat].contains(x) {
        false
    }
    r(List.nil[Nat])
    forall(a: Nat, t: List[Nat]) {
        if r(t) {
            if List.cons(a, t).contains(x) {
                if a = x {
                    x = a
                    lte_add_left(a, Nat.zero, sum(t))
                    Nat.zero <= sum(t) implies a + Nat.zero <= a + sum(t)
                    zero_lte(sum(t))
                    Nat.zero <= sum(t)
                    a + Nat.zero <= a + sum(t)
                    a + Nat.zero = a
                    a <= a + sum(t)
                    x <= a + sum(t)
                    sum(List.cons(a, t)) = a + sum(t)
                    x <= sum(List.cons(a, t))
                }
                if not a = x {
                    List.cons(a, t).contains(x) implies t.contains(x)
                    t.contains(x)
                    r(t)
                    t.contains(x) implies x <= sum(t)
                    x <= sum(t)
                    lte_add_left(sum(t), Nat.zero, a)
                    Nat.zero <= a implies sum(t) + Nat.zero <= sum(t) + a
                    zero_lte(a)
                    Nat.zero <= a
                    sum(t) + Nat.zero <= sum(t) + a
                    sum(t) + Nat.zero = sum(t)
                    sum(t) <= sum(t) + a
                    add_comm(sum(t), a)
                    sum(t) + a = a + sum(t)
                    sum(t) <= a + sum(t)
                    lte_trans(x, sum(t), a + sum(t))
                    x <= a + sum(t)
                    sum(List.cons(a, t)) = a + sum(t)
                    x <= sum(List.cons(a, t))
                }
                x <= sum(List.cons(a, t))
            }
            r(List.cons(a, t))
        }
    }
    List.induction(function(xs: List[Nat]) { r(xs) })
    r(l)
}

/// The set of partitions of `n` is finite: every partition of `n` has length
/// at most `n` and all its parts lie below `n`, so it occurs among the lists
/// of length at most `n` over `n.suc.range`.
theorem partition_set_finite(n: Nat) {
    partition_set(n).is_finite
} by {
    forall(parts: List[Nat]) {
        if partition_set(n).contains(parts) {
            partition_set_contains_iff(n, parts)
            partition_set(n).contains(parts) = is_partition(parts, n)
            is_partition(parts, n)
            is_partition_sum(parts, n)
            sum(parts) = n
            is_partition_all_positive(parts, n)
            all_positive(parts)
            sum_ge_length_all_positive(parts)
            all_positive(parts) implies parts.length <= sum(parts)
            parts.length <= sum(parts)
            parts.length <= n
            forall(x: Nat) {
                if parts.contains(x) {
                    contains_le_sum(parts, x)
                    parts.contains(x) implies x <= sum(parts)
                    x <= sum(parts)
                    x <= n
                    lt_suc(n)
                    n < n.suc
                    lte_and_lt(x, n, n.suc)
                    x <= n and n < n.suc implies x < n.suc
                    x < n.suc
                    range_contains_of_lt(n.suc, x)
                    x < n.suc implies n.suc.range.contains(x)
                    n.suc.range.contains(x)
                }
            }
            forall(x: Nat) { parts.contains(x) implies n.suc.range.contains(x) }
            parts.length <= n and (forall(x: Nat) { parts.contains(x) implies n.suc.range.contains(x) })
            all_lengths_contains(n.suc.range, parts, n)
            parts.length <= n and (forall(x: Nat) { parts.contains(x) implies n.suc.range.contains(x) }) implies all_lengths(n.suc.range, n).contains(parts)
            all_lengths(n.suc.range, n).contains(parts)
        }
    }
    forall(x: List[Nat]) {
        partition_set(n).contains(x) implies all_lengths(n.suc.range, n).contains(x)
    }
    exists(superset: List[List[Nat]]) {
        forall(x: List[Nat]) {
            partition_set(n).contains(x) implies superset.contains(x)
        }
    }
    finite_constraint(partition_set(n).contains)
    partition_set(n).is_finite
}

/// The number of partitions of `n`, i.e. the cardinality of the set of
/// partitions of `n`.
let p(n: Nat) -> result: Nat satisfy {
    partition_set(n).cardinality_is(result)
} by {
    partition_set_finite(n)
    partition_set(n).is_finite
    cardinality_always_exists(partition_set(n))
    partition_set(n).is_finite implies exists(k: Nat) { partition_set(n).cardinality_is(k) }
    exists(k: Nat) { partition_set(n).cardinality_is(k) }
    let k: Nat satisfy { partition_set(n).cardinality_is(k) }
}

// ---------------------------------------------------------------------------
// The small values of p.
// ---------------------------------------------------------------------------

/// The empty list is the only partition of zero, so p(0) = 1.
theorem p_zero {
    p(Nat.0) = Nat.1
} by {
    partition_set(Nat.0).cardinality_is(p(Nat.0))
    contains_set_intro(List.singleton[List[Nat]](List.nil[Nat]), partition_set(Nat.0))
    forall(x: List[Nat]) {
        if partition_set(Nat.0).contains(x) {
            partition_set_contains_iff(Nat.0, x)
            partition_set(Nat.0).contains(x) = is_partition(x, Nat.0)
            is_partition(x, Nat.0)
            partition_zero_char(x)
            is_partition(x, Nat.0) = (x = List.nil[Nat])
            x = List.nil[Nat]
            List.singleton[List[Nat]](List.nil[Nat]).contains(x)
        }
    }
    forall(x: List[Nat]) { partition_set(Nat.0).contains(x) implies List.singleton[List[Nat]](List.nil[Nat]).contains(x) }
    List.singleton[List[Nat]](List.nil[Nat]).contains_set(partition_set(Nat.0))
    partition_zero_char(List.nil[Nat])
    is_partition(List.nil[Nat], Nat.0) = (List.nil[Nat] = List.nil[Nat])
    List.nil[Nat] = List.nil[Nat]
    is_partition(List.nil[Nat], Nat.0) = true
    partition_set_contains_iff(Nat.0, List.nil[Nat])
    partition_set(Nat.0).contains(List.nil[Nat]) = is_partition(List.nil[Nat], Nat.0)
    partition_set(Nat.0).contains(List.nil[Nat]) = true
    partition_set(Nat.0).contains(List.nil[Nat])
    List.singleton[List[Nat]](List.nil[Nat]).filter(partition_set(Nat.0).contains) =
        List.cons(List.nil[Nat], List.nil[List[Nat]].filter(partition_set(Nat.0).contains))
    List.nil[List[Nat]].filter(partition_set(Nat.0).contains) = List.nil[List[Nat]]
    List.singleton[List[Nat]](List.nil[Nat]).filter(partition_set(Nat.0).contains) =
        List.cons(List.nil[Nat], List.nil[List[Nat]])
    List.cons(List.nil[Nat], List.nil[List[Nat]]) = List.singleton[List[Nat]](List.nil[Nat])
    List.singleton[List[Nat]](List.nil[Nat]).filter(partition_set(Nat.0).contains) =
        List.singleton[List[Nat]](List.nil[Nat])
    singleton_unique(List.nil[Nat])
    List.singleton[List[Nat]](List.nil[Nat]).unique = List.singleton[List[Nat]](List.nil[Nat])
    List.singleton[List[Nat]](List.nil[Nat]).length = List.nil[List[Nat]].length.suc
    List.nil[List[Nat]].length = Nat.0
    List.singleton[List[Nat]](List.nil[Nat]).length = Nat.0.suc
    Nat.0.suc = Nat.1
    List.singleton[List[Nat]](List.nil[Nat]).length = Nat.1
    List.singleton[List[Nat]](List.nil[Nat]).filter(partition_set(Nat.0).contains).unique =
        List.singleton[List[Nat]](List.nil[Nat])
    List.singleton[List[Nat]](List.nil[Nat]).filter(partition_set(Nat.0).contains).unique.length = Nat.1
    List.singleton[List[Nat]](List.nil[Nat]).contains_set(partition_set(Nat.0)) and
        List.singleton[List[Nat]](List.nil[Nat]).filter(partition_set(Nat.0).contains).unique.length = Nat.1
    exists(containing_list: List[List[Nat]]) {
        containing_list.contains_set(partition_set(Nat.0)) and
            containing_list.filter(partition_set(Nat.0).contains).unique.length = Nat.1
    }
    partition_set(Nat.0).cardinality_is(Nat.1)
    cardinality_is_well_defined(partition_set(Nat.0), p(Nat.0), Nat.1)
    partition_set(Nat.0).cardinality_is(p(Nat.0)) and partition_set(Nat.0).cardinality_is(Nat.1) implies p(Nat.0) = Nat.1
    p(Nat.0) = Nat.1
}

/// The explicit list of the partitions of one.
let partitions_one: List[List[Nat]] = List.cons(List.cons(Nat.1, List.nil[Nat]), List.nil[List[Nat]])

/// The explicit list of the partitions of two.
let partitions_two: List[List[Nat]] =
    List.cons(List.cons(Nat.2, List.nil[Nat]),
        List.cons(List.cons(Nat.1, List.cons(Nat.1, List.nil[Nat])), List.nil[List[Nat]]))

/// The explicit list of the partitions of three.
let partitions_three: List[List[Nat]] =
    List.cons(List.cons(Nat.3, List.nil[Nat]),
        List.cons(List.cons(Nat.2, List.cons(Nat.1, List.nil[Nat])),
            List.cons(List.cons(Nat.1, List.cons(Nat.1, List.cons(Nat.1, List.nil[Nat]))),
                List.nil[List[Nat]])))

/// The explicit list of the partitions of four.
let partitions_four: List[List[Nat]] =
    List.cons(List.cons(Nat.4, List.nil[Nat]),
        List.cons(List.cons(Nat.3, List.cons(Nat.1, List.nil[Nat])),
            List.cons(List.cons(Nat.2, List.cons(Nat.2, List.nil[Nat])),
                List.cons(List.cons(Nat.2, List.cons(Nat.1, List.cons(Nat.1, List.nil[Nat]))),
                    List.cons(List.cons(Nat.1, List.cons(Nat.1, List.cons(Nat.1, List.cons(Nat.1, List.nil[Nat])))),
                        List.nil[List[Nat]])))))

/// A two-element list contains its last element.
theorem contains_list_2(x: List[Nat], a: List[Nat], b: List[Nat]) {
    x = b implies List.cons(a, List.cons(b, List.nil[List[Nat]])).contains(x)
} by {
    if x = b {
        cons_contains_head(b, List.nil[List[Nat]])
        List.cons(b, List.nil[List[Nat]]).contains(b)
        List.cons(b, List.nil[List[Nat]]).contains(x)
        cons_contains_of_tail_contains(a, List.cons(b, List.nil[List[Nat]]), x)
        List.cons(b, List.nil[List[Nat]]).contains(x) implies List.cons(a, List.cons(b, List.nil[List[Nat]])).contains(x)
        List.cons(a, List.cons(b, List.nil[List[Nat]])).contains(x)
    }
}

/// A three-element list contains its last element.
theorem contains_list_3(x: List[Nat], a: List[Nat], b: List[Nat], c: List[Nat]) {
    x = c implies List.cons(a, List.cons(b, List.cons(c, List.nil[List[Nat]]))).contains(x)
} by {
    if x = c {
        cons_contains_head(c, List.nil[List[Nat]])
        List.cons(c, List.nil[List[Nat]]).contains(c)
        List.cons(c, List.nil[List[Nat]]).contains(x)
        cons_contains_of_tail_contains(b, List.cons(c, List.nil[List[Nat]]), x)
        List.cons(c, List.nil[List[Nat]]).contains(x) implies List.cons(b, List.cons(c, List.nil[List[Nat]])).contains(x)
        List.cons(b, List.cons(c, List.nil[List[Nat]])).contains(x)
        cons_contains_of_tail_contains(a, List.cons(b, List.cons(c, List.nil[List[Nat]])), x)
        List.cons(b, List.cons(c, List.nil[List[Nat]])).contains(x) implies List.cons(a, List.cons(b, List.cons(c, List.nil[List[Nat]]))).contains(x)
        List.cons(a, List.cons(b, List.cons(c, List.nil[List[Nat]]))).contains(x)
    }
}

/// A four-element list contains its last element.
theorem contains_list_4(x: List[Nat], a: List[Nat], b: List[Nat], c: List[Nat], d: List[Nat]) {
    x = d implies List.cons(a, List.cons(b, List.cons(c, List.cons(d, List.nil[List[Nat]])))).contains(x)
} by {
    if x = d {
        cons_contains_head(d, List.nil[List[Nat]])
        List.cons(d, List.nil[List[Nat]]).contains(d)
        List.cons(d, List.nil[List[Nat]]).contains(x)
        cons_contains_of_tail_contains(c, List.cons(d, List.nil[List[Nat]]), x)
        List.cons(c, List.cons(d, List.nil[List[Nat]])).contains(x)
        cons_contains_of_tail_contains(b, List.cons(c, List.cons(d, List.nil[List[Nat]])), x)
        List.cons(b, List.cons(c, List.cons(d, List.nil[List[Nat]]))).contains(x)
        cons_contains_of_tail_contains(a, List.cons(b, List.cons(c, List.cons(d, List.nil[List[Nat]]))), x)
        List.cons(a, List.cons(b, List.cons(c, List.cons(d, List.nil[List[Nat]])))).contains(x)
    }
}

/// Every partition of one occurs in the explicit list.
theorem partitions_one_contains(l: List[Nat]) {
    is_partition(l, Nat.1) implies partitions_one.contains(l)
} by {
    if is_partition(l, Nat.1) {
        partition_one_char(l)
        is_partition(l, Nat.1) = (l = List.cons(Nat.1, List.nil[Nat]))
        l = List.cons(Nat.1, List.nil[Nat])
        cons_contains_head(List.cons(Nat.1, List.nil[Nat]), List.nil[List[Nat]])
        List.cons(List.cons(Nat.1, List.nil[Nat]), List.nil[List[Nat]]).contains(List.cons(Nat.1, List.nil[Nat]))
        partitions_one.contains(List.cons(Nat.1, List.nil[Nat]))
        partitions_one.contains(l)
    }
}

/// Every partition of two occurs in the explicit list.
theorem partitions_two_contains(l: List[Nat]) {
    is_partition(l, Nat.2) implies partitions_two.contains(l)
} by {
    if is_partition(l, Nat.2) {
        partition_two_char(l)
        is_partition(l, Nat.2) = (l = List.cons(Nat.2, List.nil[Nat]) or l = List.cons(Nat.1, List.cons(Nat.1, List.nil[Nat])))
        l = List.cons(Nat.2, List.nil[Nat]) or l = List.cons(Nat.1, List.cons(Nat.1, List.nil[Nat]))
        if l = List.cons(Nat.2, List.nil[Nat]) {
            partitions_two.contains(List.cons(Nat.2, List.nil[Nat]))
            partitions_two.contains(l)
        }
        if not l = List.cons(Nat.2, List.nil[Nat]) {
            l = List.cons(Nat.1, List.cons(Nat.1, List.nil[Nat]))
            partitions_two.contains(List.cons(Nat.1, List.cons(Nat.1, List.nil[Nat])))
            partitions_two.contains(l)
        }
        partitions_two.contains(l)
    }
}

/// Every partition of three occurs in the explicit list.
theorem partitions_three_contains(l: List[Nat]) {
    is_partition(l, Nat.3) implies partitions_three.contains(l)
} by {
    if is_partition(l, Nat.3) {
        partition_three_char(l)
        is_partition(l, Nat.3) = (l = List.cons(Nat.3, List.nil[Nat]) or l = List.cons(Nat.2, List.cons(Nat.1, List.nil[Nat])) or l = List.cons(Nat.1, List.cons(Nat.1, List.cons(Nat.1, List.nil[Nat]))))
        l = List.cons(Nat.3, List.nil[Nat]) or l = List.cons(Nat.2, List.cons(Nat.1, List.nil[Nat])) or l = List.cons(Nat.1, List.cons(Nat.1, List.cons(Nat.1, List.nil[Nat])))
        if l = List.cons(Nat.3, List.nil[Nat]) {
            cons_contains_head(List.cons(Nat.3, List.nil[Nat]),
                List.cons(List.cons(Nat.2, List.cons(Nat.1, List.nil[Nat])),
                    List.cons(List.cons(Nat.1, List.cons(Nat.1, List.cons(Nat.1, List.nil[Nat]))), List.nil[List[Nat]])))
            List.cons(List.cons(Nat.3, List.nil[Nat]),
                List.cons(List.cons(Nat.2, List.cons(Nat.1, List.nil[Nat])),
                    List.cons(List.cons(Nat.1, List.cons(Nat.1, List.cons(Nat.1, List.nil[Nat]))), List.nil[List[Nat]]))).contains(List.cons(Nat.3, List.nil[Nat]))
            partitions_three.contains(List.cons(Nat.3, List.nil[Nat]))
            partitions_three.contains(l)
        }
        if not l = List.cons(Nat.3, List.nil[Nat]) {
            if l = List.cons(Nat.2, List.cons(Nat.1, List.nil[Nat])) {
                partitions_three.contains(List.cons(Nat.2, List.cons(Nat.1, List.nil[Nat])))
                partitions_three.contains(l)
            }
            if not l = List.cons(Nat.2, List.cons(Nat.1, List.nil[Nat])) {
                l = List.cons(Nat.1, List.cons(Nat.1, List.cons(Nat.1, List.nil[Nat])))
                contains_list_3(List.cons(Nat.1, List.cons(Nat.1, List.cons(Nat.1, List.nil[Nat]))),
                    List.cons(Nat.3, List.nil[Nat]),
                    List.cons(Nat.2, List.cons(Nat.1, List.nil[Nat])),
                    List.cons(Nat.1, List.cons(Nat.1, List.cons(Nat.1, List.nil[Nat]))))
                List.cons(Nat.1, List.cons(Nat.1, List.cons(Nat.1, List.nil[Nat]))) =
                    List.cons(Nat.1, List.cons(Nat.1, List.cons(Nat.1, List.nil[Nat])))
                List.cons(List.cons(Nat.3, List.nil[Nat]),
                    List.cons(List.cons(Nat.2, List.cons(Nat.1, List.nil[Nat])),
                        List.cons(List.cons(Nat.1, List.cons(Nat.1, List.cons(Nat.1, List.nil[Nat]))), List.nil[List[Nat]]))).contains(List.cons(Nat.1, List.cons(Nat.1, List.cons(Nat.1, List.nil[Nat]))))
                partitions_three.contains(List.cons(Nat.1, List.cons(Nat.1, List.cons(Nat.1, List.nil[Nat]))))
                partitions_three.contains(l)
            }
            partitions_three.contains(l)
        }
        partitions_three.contains(l)
    }
}

/// A two-element list of distinct elements is unique.
theorem pair_is_unique(a: List[Nat], b: List[Nat]) {
    a != b implies List.cons(a, List.cons(b, List.nil[List[Nat]])).is_unique
} by {
    if a != b {
        singleton_unique(b)
        List.singleton[List[Nat]](b).is_unique
        List.cons(b, List.nil[List[Nat]]) = List.singleton[List[Nat]](b)
        List.cons(b, List.nil[List[Nat]]).is_unique
        nil_not_contains(b)
        not List.nil[List[Nat]].contains(b)
        cons_unique_of_tail_unique_not_contains(b, List.nil[List[Nat]])
        List.nil[List[Nat]].is_unique and not List.nil[List[Nat]].contains(b) implies List.cons(b, List.nil[List[Nat]]).is_unique
        List.nil[List[Nat]].is_unique
        List.nil[List[Nat]].is_unique and not List.nil[List[Nat]].contains(b)
        List.cons(b, List.nil[List[Nat]]).is_unique
        cons_contains_eq(b, List.nil[List[Nat]], a)
        List.cons(b, List.nil[List[Nat]]).contains(a) =
            (b = a or List.nil[List[Nat]].contains(a))
        b != a
        not List.nil[List[Nat]].contains(a)
        not List.cons(b, List.nil[List[Nat]]).contains(a)
        cons_unique_of_tail_unique_not_contains(a, List.cons(b, List.nil[List[Nat]]))
        List.cons(b, List.nil[List[Nat]]).is_unique and not List.cons(b, List.nil[List[Nat]]).contains(a) implies List.cons(a, List.cons(b, List.nil[List[Nat]])).is_unique
        List.cons(b, List.nil[List[Nat]]).is_unique and not List.cons(b, List.nil[List[Nat]]).contains(a)
        List.cons(a, List.cons(b, List.nil[List[Nat]])).is_unique
    }
}

/// The singleton 1 is the only partition of one, so p(1) = 1.
theorem p_one {
    p(Nat.1) = Nat.1
} by {
    partition_set(Nat.1).cardinality_is(p(Nat.1))
    contains_set_intro(partitions_one, partition_set(Nat.1))
    forall(l: List[Nat]) {
        if partition_set(Nat.1).contains(l) {
            partition_set_contains_iff(Nat.1, l)
            partition_set(Nat.1).contains(l) = is_partition(l, Nat.1)
            is_partition(l, Nat.1)
            partitions_one_contains(l)
            is_partition(l, Nat.1) implies partitions_one.contains(l)
            partitions_one.contains(l)
        }
    }
    forall(l: List[Nat]) { partition_set(Nat.1).contains(l) implies partitions_one.contains(l) }
    partitions_one.contains_set(partition_set(Nat.1))
    partition_one_char(List.cons(Nat.1, List.nil[Nat]))
    is_partition(List.cons(Nat.1, List.nil[Nat]), Nat.1) = (List.cons(Nat.1, List.nil[Nat]) = List.cons(Nat.1, List.nil[Nat]))
    List.cons(Nat.1, List.nil[Nat]) = List.cons(Nat.1, List.nil[Nat])
    is_partition(List.cons(Nat.1, List.nil[Nat]), Nat.1) = true
    partition_set_contains_iff(Nat.1, List.cons(Nat.1, List.nil[Nat]))
    partition_set(Nat.1).contains(List.cons(Nat.1, List.nil[Nat])) = is_partition(List.cons(Nat.1, List.nil[Nat]), Nat.1)
    partition_set(Nat.1).contains(List.cons(Nat.1, List.nil[Nat])) = true
    partition_set(Nat.1).contains(List.cons(Nat.1, List.nil[Nat]))
    partitions_one = List.cons(List.cons(Nat.1, List.nil[Nat]), List.nil[List[Nat]])
    List.cons(List.cons(Nat.1, List.nil[Nat]), List.nil[List[Nat]]).filter(partition_set(Nat.1).contains) =
        List.cons(List.cons(Nat.1, List.nil[Nat]), List.nil[List[Nat]].filter(partition_set(Nat.1).contains))
    List.nil[List[Nat]].filter(partition_set(Nat.1).contains) = List.nil[List[Nat]]
    List.cons(List.cons(Nat.1, List.nil[Nat]), List.nil[List[Nat]]).filter(partition_set(Nat.1).contains) =
        List.cons(List.cons(Nat.1, List.nil[Nat]), List.nil[List[Nat]])
    partitions_one.filter(partition_set(Nat.1).contains) = partitions_one
    singleton_unique(List.cons(Nat.1, List.nil[Nat]))
    partitions_one.unique = partitions_one
    partitions_one.length = List.nil[List[Nat]].length.suc
    List.nil[List[Nat]].length = Nat.0
    partitions_one.length = Nat.0.suc
    Nat.0.suc = Nat.1
    partitions_one.length = Nat.1
    partitions_one.filter(partition_set(Nat.1).contains).unique = partitions_one
    partitions_one.filter(partition_set(Nat.1).contains).unique.length = Nat.1
    partitions_one.contains_set(partition_set(Nat.1)) and
        partitions_one.filter(partition_set(Nat.1).contains).unique.length = Nat.1
    exists(containing_list: List[List[Nat]]) {
        containing_list.contains_set(partition_set(Nat.1)) and
            containing_list.filter(partition_set(Nat.1).contains).unique.length = Nat.1
    }
    partition_set(Nat.1).cardinality_is(Nat.1)
    cardinality_is_well_defined(partition_set(Nat.1), p(Nat.1), Nat.1)
    partition_set(Nat.1).cardinality_is(p(Nat.1)) and partition_set(Nat.1).cardinality_is(Nat.1) implies p(Nat.1) = Nat.1
    p(Nat.1) = Nat.1
}

/// A three-element list of pairwise distinct elements is unique.
theorem triple_is_unique(a: List[Nat], b: List[Nat], c: List[Nat]) {
    a != b and a != c and b != c implies List.cons(a, List.cons(b, List.cons(c, List.nil[List[Nat]]))).is_unique
} by {
    if a != b and a != c and b != c {
        pair_is_unique(b, c)
        b != c implies List.cons(b, List.cons(c, List.nil[List[Nat]])).is_unique
        List.cons(b, List.cons(c, List.nil[List[Nat]])).is_unique
        cons_contains_eq(b, List.cons(c, List.nil[List[Nat]]), a)
        List.cons(b, List.cons(c, List.nil[List[Nat]])).contains(a) =
            (b = a or List.cons(c, List.nil[List[Nat]]).contains(a))
        b != a
        cons_contains_eq(c, List.nil[List[Nat]], a)
        List.cons(c, List.nil[List[Nat]]).contains(a) =
            (c = a or List.nil[List[Nat]].contains(a))
        c != a
        not List.nil[List[Nat]].contains(a)
        not List.cons(c, List.nil[List[Nat]]).contains(a)
        not List.cons(b, List.cons(c, List.nil[List[Nat]])).contains(a)
        cons_unique_of_tail_unique_not_contains(a, List.cons(b, List.cons(c, List.nil[List[Nat]])))
        List.cons(b, List.cons(c, List.nil[List[Nat]])).is_unique and not List.cons(b, List.cons(c, List.nil[List[Nat]])).contains(a) implies List.cons(a, List.cons(b, List.cons(c, List.nil[List[Nat]]))).is_unique
        List.cons(b, List.cons(c, List.nil[List[Nat]])).is_unique and not List.cons(b, List.cons(c, List.nil[List[Nat]])).contains(a)
        List.cons(a, List.cons(b, List.cons(c, List.nil[List[Nat]]))).is_unique
    }
}

/// The partitions of two are 2 and 1 + 1, so p(2) = 2.
theorem p_two {
    p(Nat.2) = Nat.2
} by {
    partition_set(Nat.2).cardinality_is(p(Nat.2))
    contains_set_intro(partitions_two, partition_set(Nat.2))
    forall(l: List[Nat]) {
        if partition_set(Nat.2).contains(l) {
            partition_set_contains_iff(Nat.2, l)
            partition_set(Nat.2).contains(l) = is_partition(l, Nat.2)
            is_partition(l, Nat.2)
            partitions_two_contains(l)
            is_partition(l, Nat.2) implies partitions_two.contains(l)
            partitions_two.contains(l)
        }
    }
    forall(l: List[Nat]) { partition_set(Nat.2).contains(l) implies partitions_two.contains(l) }
    partitions_two.contains_set(partition_set(Nat.2))
    partition_two_char(List.cons(Nat.2, List.nil[Nat]))
    is_partition(List.cons(Nat.2, List.nil[Nat]), Nat.2) =
        (List.cons(Nat.2, List.nil[Nat]) = List.cons(Nat.2, List.nil[Nat]) or List.cons(Nat.2, List.nil[Nat]) = List.cons(Nat.1, List.cons(Nat.1, List.nil[Nat])))
    List.cons(Nat.2, List.nil[Nat]) = List.cons(Nat.2, List.nil[Nat])
    is_partition(List.cons(Nat.2, List.nil[Nat]), Nat.2) = true
    partition_set_contains_iff(Nat.2, List.cons(Nat.2, List.nil[Nat]))
    partition_set(Nat.2).contains(List.cons(Nat.2, List.nil[Nat])) = is_partition(List.cons(Nat.2, List.nil[Nat]), Nat.2)
    partition_set(Nat.2).contains(List.cons(Nat.2, List.nil[Nat])) = true
    partition_set(Nat.2).contains(List.cons(Nat.2, List.nil[Nat]))
    partition_two_char(List.cons(Nat.1, List.cons(Nat.1, List.nil[Nat])))
    is_partition(List.cons(Nat.1, List.cons(Nat.1, List.nil[Nat])), Nat.2) =
        (List.cons(Nat.1, List.cons(Nat.1, List.nil[Nat])) = List.cons(Nat.2, List.nil[Nat]) or List.cons(Nat.1, List.cons(Nat.1, List.nil[Nat])) = List.cons(Nat.1, List.cons(Nat.1, List.nil[Nat])))
    List.cons(Nat.1, List.cons(Nat.1, List.nil[Nat])) = List.cons(Nat.1, List.cons(Nat.1, List.nil[Nat]))
    is_partition(List.cons(Nat.1, List.cons(Nat.1, List.nil[Nat])), Nat.2) = true
    partition_set_contains_iff(Nat.2, List.cons(Nat.1, List.cons(Nat.1, List.nil[Nat])))
    partition_set(Nat.2).contains(List.cons(Nat.1, List.cons(Nat.1, List.nil[Nat]))) = is_partition(List.cons(Nat.1, List.cons(Nat.1, List.nil[Nat])), Nat.2)
    partition_set(Nat.2).contains(List.cons(Nat.1, List.cons(Nat.1, List.nil[Nat]))) = true
    partition_set(Nat.2).contains(List.cons(Nat.1, List.cons(Nat.1, List.nil[Nat])))
    partitions_two = List.cons(List.cons(Nat.2, List.nil[Nat]),
        List.cons(List.cons(Nat.1, List.cons(Nat.1, List.nil[Nat])), List.nil[List[Nat]]))
    List.cons(List.cons(Nat.2, List.nil[Nat]),
        List.cons(List.cons(Nat.1, List.cons(Nat.1, List.nil[Nat])), List.nil[List[Nat]])).filter(partition_set(Nat.2).contains) =
        List.cons(List.cons(Nat.2, List.nil[Nat]),
            List.cons(List.cons(Nat.1, List.cons(Nat.1, List.nil[Nat])), List.nil[List[Nat]]).filter(partition_set(Nat.2).contains))
    List.cons(List.cons(Nat.1, List.cons(Nat.1, List.nil[Nat])), List.nil[List[Nat]]).filter(partition_set(Nat.2).contains) =
        List.cons(List.cons(Nat.1, List.cons(Nat.1, List.nil[Nat])), List.nil[List[Nat]].filter(partition_set(Nat.2).contains))
    List.nil[List[Nat]].filter(partition_set(Nat.2).contains) = List.nil[List[Nat]]
    List.cons(List.cons(Nat.1, List.cons(Nat.1, List.nil[Nat])), List.nil[List[Nat]]).filter(partition_set(Nat.2).contains) =
        List.cons(List.cons(Nat.1, List.cons(Nat.1, List.nil[Nat])), List.nil[List[Nat]])
    List.cons(List.cons(Nat.2, List.nil[Nat]),
        List.cons(List.cons(Nat.1, List.cons(Nat.1, List.nil[Nat])), List.nil[List[Nat]])).filter(partition_set(Nat.2).contains) =
        List.cons(List.cons(Nat.2, List.nil[Nat]),
            List.cons(List.cons(Nat.1, List.cons(Nat.1, List.nil[Nat])), List.nil[List[Nat]]))
    partitions_two.filter(partition_set(Nat.2).contains) = partitions_two
    List.cons(Nat.2, List.nil[Nat]) != List.cons(Nat.1, List.cons(Nat.1, List.nil[Nat]))
    pair_is_unique(List.cons(Nat.2, List.nil[Nat]), List.cons(Nat.1, List.cons(Nat.1, List.nil[Nat])))
    List.cons(Nat.2, List.nil[Nat]) != List.cons(Nat.1, List.cons(Nat.1, List.nil[Nat])) implies List.cons(List.cons(Nat.2, List.nil[Nat]), List.cons(List.cons(Nat.1, List.cons(Nat.1, List.nil[Nat])), List.nil[List[Nat]])).is_unique
    List.cons(List.cons(Nat.2, List.nil[Nat]), List.cons(List.cons(Nat.1, List.cons(Nat.1, List.nil[Nat])), List.nil[List[Nat]])).is_unique
    partitions_two = List.cons(List.cons(Nat.2, List.nil[Nat]),
        List.cons(List.cons(Nat.1, List.cons(Nat.1, List.nil[Nat])), List.nil[List[Nat]]))
    partitions_two.is_unique
    partitions_two.unique = partitions_two
    partitions_two.length = List.cons(List.cons(Nat.1, List.cons(Nat.1, List.nil[Nat])), List.nil[List[Nat]]).length.suc
    List.cons(List.cons(Nat.1, List.cons(Nat.1, List.nil[Nat])), List.nil[List[Nat]]).length = List.nil[List[Nat]].length.suc
    List.nil[List[Nat]].length = Nat.0
    List.cons(List.cons(Nat.1, List.cons(Nat.1, List.nil[Nat])), List.nil[List[Nat]]).length = Nat.0.suc
    partitions_two.length = Nat.0.suc.suc
    Nat.0.suc.suc = Nat.2
    partitions_two.length = Nat.2
    partitions_two.filter(partition_set(Nat.2).contains).unique = partitions_two
    partitions_two.filter(partition_set(Nat.2).contains).unique.length = Nat.2
    partitions_two.contains_set(partition_set(Nat.2)) and
        partitions_two.filter(partition_set(Nat.2).contains).unique.length = Nat.2
    exists(containing_list: List[List[Nat]]) {
        containing_list.contains_set(partition_set(Nat.2)) and
            containing_list.filter(partition_set(Nat.2).contains).unique.length = Nat.2
    }
    partition_set(Nat.2).cardinality_is(Nat.2)
    cardinality_is_well_defined(partition_set(Nat.2), p(Nat.2), Nat.2)
    partition_set(Nat.2).cardinality_is(p(Nat.2)) and partition_set(Nat.2).cardinality_is(Nat.2) implies p(Nat.2) = Nat.2
    p(Nat.2) = Nat.2
}

/// A four-element list of pairwise distinct elements is unique.
theorem quad_is_unique(a: List[Nat], b: List[Nat], c: List[Nat], d: List[Nat]) {
    a != b and a != c and a != d and b != c and b != d and c != d implies List.cons(a, List.cons(b, List.cons(c, List.cons(d, List.nil[List[Nat]])))).is_unique
} by {
    if a != b and a != c and a != d and b != c and b != d and c != d {
        triple_is_unique(b, c, d)
        b != c and b != d and c != d implies List.cons(b, List.cons(c, List.cons(d, List.nil[List[Nat]]))).is_unique
        b != c and b != d and c != d
        List.cons(b, List.cons(c, List.cons(d, List.nil[List[Nat]]))).is_unique
        cons_contains_eq(b, List.cons(c, List.cons(d, List.nil[List[Nat]])), a)
        List.cons(b, List.cons(c, List.cons(d, List.nil[List[Nat]]))).contains(a) =
            (b = a or List.cons(c, List.cons(d, List.nil[List[Nat]])).contains(a))
        b != a
        cons_contains_eq(c, List.cons(d, List.nil[List[Nat]]), a)
        List.cons(c, List.cons(d, List.nil[List[Nat]])).contains(a) =
            (c = a or List.cons(d, List.nil[List[Nat]]).contains(a))
        c != a
        cons_contains_eq(d, List.nil[List[Nat]], a)
        List.cons(d, List.nil[List[Nat]]).contains(a) =
            (d = a or List.nil[List[Nat]].contains(a))
        d != a
        not List.nil[List[Nat]].contains(a)
        not List.cons(d, List.nil[List[Nat]]).contains(a)
        not List.cons(c, List.cons(d, List.nil[List[Nat]])).contains(a)
        not List.cons(b, List.cons(c, List.cons(d, List.nil[List[Nat]]))).contains(a)
        cons_unique_of_tail_unique_not_contains(a, List.cons(b, List.cons(c, List.cons(d, List.nil[List[Nat]]))))
        List.cons(b, List.cons(c, List.cons(d, List.nil[List[Nat]]))).is_unique and not List.cons(b, List.cons(c, List.cons(d, List.nil[List[Nat]]))).contains(a) implies List.cons(a, List.cons(b, List.cons(c, List.cons(d, List.nil[List[Nat]])))).is_unique
        List.cons(b, List.cons(c, List.cons(d, List.nil[List[Nat]]))).is_unique and not List.cons(b, List.cons(c, List.cons(d, List.nil[List[Nat]]))).contains(a)
        List.cons(a, List.cons(b, List.cons(c, List.cons(d, List.nil[List[Nat]])))).is_unique
    }
}

/// The partitions of three are 3, 2 + 1 and 1 + 1 + 1, so p(3) = 3.
theorem p_three {
    p(Nat.3) = Nat.3
} by {
    partition_set(Nat.3).cardinality_is(p(Nat.3))
    contains_set_intro(partitions_three, partition_set(Nat.3))
    forall(l: List[Nat]) {
        if partition_set(Nat.3).contains(l) {
            partition_set_contains_iff(Nat.3, l)
            partition_set(Nat.3).contains(l) = is_partition(l, Nat.3)
            is_partition(l, Nat.3)
            partitions_three_contains(l)
            is_partition(l, Nat.3) implies partitions_three.contains(l)
            partitions_three.contains(l)
        }
    }
    forall(l: List[Nat]) { partition_set(Nat.3).contains(l) implies partitions_three.contains(l) }
    partitions_three.contains_set(partition_set(Nat.3))
    partition_three_char(List.cons(Nat.3, List.nil[Nat]))
    is_partition(List.cons(Nat.3, List.nil[Nat]), Nat.3) =
        (List.cons(Nat.3, List.nil[Nat]) = List.cons(Nat.3, List.nil[Nat]) or List.cons(Nat.3, List.nil[Nat]) = List.cons(Nat.2, List.cons(Nat.1, List.nil[Nat])) or List.cons(Nat.3, List.nil[Nat]) = List.cons(Nat.1, List.cons(Nat.1, List.cons(Nat.1, List.nil[Nat]))))
    List.cons(Nat.3, List.nil[Nat]) = List.cons(Nat.3, List.nil[Nat])
    is_partition(List.cons(Nat.3, List.nil[Nat]), Nat.3) = true
    partition_set_contains_iff(Nat.3, List.cons(Nat.3, List.nil[Nat]))
    partition_set(Nat.3).contains(List.cons(Nat.3, List.nil[Nat])) = is_partition(List.cons(Nat.3, List.nil[Nat]), Nat.3)
    partition_set(Nat.3).contains(List.cons(Nat.3, List.nil[Nat])) = true
    partition_set(Nat.3).contains(List.cons(Nat.3, List.nil[Nat]))
    partition_three_char(List.cons(Nat.2, List.cons(Nat.1, List.nil[Nat])))
    is_partition(List.cons(Nat.2, List.cons(Nat.1, List.nil[Nat])), Nat.3) =
        (List.cons(Nat.2, List.cons(Nat.1, List.nil[Nat])) = List.cons(Nat.3, List.nil[Nat]) or List.cons(Nat.2, List.cons(Nat.1, List.nil[Nat])) = List.cons(Nat.2, List.cons(Nat.1, List.nil[Nat])) or List.cons(Nat.2, List.cons(Nat.1, List.nil[Nat])) = List.cons(Nat.1, List.cons(Nat.1, List.cons(Nat.1, List.nil[Nat]))))
    List.cons(Nat.2, List.cons(Nat.1, List.nil[Nat])) = List.cons(Nat.2, List.cons(Nat.1, List.nil[Nat]))
    is_partition(List.cons(Nat.2, List.cons(Nat.1, List.nil[Nat])), Nat.3) = true
    partition_set_contains_iff(Nat.3, List.cons(Nat.2, List.cons(Nat.1, List.nil[Nat])))
    partition_set(Nat.3).contains(List.cons(Nat.2, List.cons(Nat.1, List.nil[Nat]))) = is_partition(List.cons(Nat.2, List.cons(Nat.1, List.nil[Nat])), Nat.3)
    partition_set(Nat.3).contains(List.cons(Nat.2, List.cons(Nat.1, List.nil[Nat]))) = true
    partition_set(Nat.3).contains(List.cons(Nat.2, List.cons(Nat.1, List.nil[Nat])))
    partition_three_char(List.cons(Nat.1, List.cons(Nat.1, List.cons(Nat.1, List.nil[Nat]))))
    is_partition(List.cons(Nat.1, List.cons(Nat.1, List.cons(Nat.1, List.nil[Nat]))), Nat.3) =
        (List.cons(Nat.1, List.cons(Nat.1, List.cons(Nat.1, List.nil[Nat]))) = List.cons(Nat.3, List.nil[Nat]) or List.cons(Nat.1, List.cons(Nat.1, List.cons(Nat.1, List.nil[Nat]))) = List.cons(Nat.2, List.cons(Nat.1, List.nil[Nat])) or List.cons(Nat.1, List.cons(Nat.1, List.cons(Nat.1, List.nil[Nat]))) = List.cons(Nat.1, List.cons(Nat.1, List.cons(Nat.1, List.nil[Nat]))))
    List.cons(Nat.1, List.cons(Nat.1, List.cons(Nat.1, List.nil[Nat]))) = List.cons(Nat.1, List.cons(Nat.1, List.cons(Nat.1, List.nil[Nat])))
    is_partition(List.cons(Nat.1, List.cons(Nat.1, List.cons(Nat.1, List.nil[Nat]))), Nat.3) = true
    partition_set_contains_iff(Nat.3, List.cons(Nat.1, List.cons(Nat.1, List.cons(Nat.1, List.nil[Nat]))))
    partition_set(Nat.3).contains(List.cons(Nat.1, List.cons(Nat.1, List.cons(Nat.1, List.nil[Nat])))) = is_partition(List.cons(Nat.1, List.cons(Nat.1, List.cons(Nat.1, List.nil[Nat]))), Nat.3)
    partition_set(Nat.3).contains(List.cons(Nat.1, List.cons(Nat.1, List.cons(Nat.1, List.nil[Nat])))) = true
    partition_set(Nat.3).contains(List.cons(Nat.1, List.cons(Nat.1, List.cons(Nat.1, List.nil[Nat]))))
    partitions_three = List.cons(List.cons(Nat.3, List.nil[Nat]),
        List.cons(List.cons(Nat.2, List.cons(Nat.1, List.nil[Nat])),
            List.cons(List.cons(Nat.1, List.cons(Nat.1, List.cons(Nat.1, List.nil[Nat]))), List.nil[List[Nat]])))
    List.cons(List.cons(Nat.3, List.nil[Nat]),
        List.cons(List.cons(Nat.2, List.cons(Nat.1, List.nil[Nat])),
            List.cons(List.cons(Nat.1, List.cons(Nat.1, List.cons(Nat.1, List.nil[Nat]))), List.nil[List[Nat]]))).filter(partition_set(Nat.3).contains) =
        List.cons(List.cons(Nat.3, List.nil[Nat]),
            List.cons(List.cons(Nat.2, List.cons(Nat.1, List.nil[Nat])),
                List.cons(List.cons(Nat.1, List.cons(Nat.1, List.cons(Nat.1, List.nil[Nat]))), List.nil[List[Nat]])).filter(partition_set(Nat.3).contains))
    List.cons(List.cons(Nat.2, List.cons(Nat.1, List.nil[Nat])),
        List.cons(List.cons(Nat.1, List.cons(Nat.1, List.cons(Nat.1, List.nil[Nat]))), List.nil[List[Nat]])).filter(partition_set(Nat.3).contains) =
        List.cons(List.cons(Nat.2, List.cons(Nat.1, List.nil[Nat])),
            List.cons(List.cons(Nat.1, List.cons(Nat.1, List.cons(Nat.1, List.nil[Nat]))), List.nil[List[Nat]]).filter(partition_set(Nat.3).contains))
    List.cons(List.cons(Nat.1, List.cons(Nat.1, List.cons(Nat.1, List.nil[Nat]))), List.nil[List[Nat]]).filter(partition_set(Nat.3).contains) =
        List.cons(List.cons(Nat.1, List.cons(Nat.1, List.cons(Nat.1, List.nil[Nat]))), List.nil[List[Nat]].filter(partition_set(Nat.3).contains))
    List.nil[List[Nat]].filter(partition_set(Nat.3).contains) = List.nil[List[Nat]]
    List.cons(List.cons(Nat.1, List.cons(Nat.1, List.cons(Nat.1, List.nil[Nat]))), List.nil[List[Nat]]).filter(partition_set(Nat.3).contains) =
        List.cons(List.cons(Nat.1, List.cons(Nat.1, List.cons(Nat.1, List.nil[Nat]))), List.nil[List[Nat]])
    List.cons(List.cons(Nat.2, List.cons(Nat.1, List.nil[Nat])),
        List.cons(List.cons(Nat.1, List.cons(Nat.1, List.cons(Nat.1, List.nil[Nat]))), List.nil[List[Nat]])).filter(partition_set(Nat.3).contains) =
        List.cons(List.cons(Nat.2, List.cons(Nat.1, List.nil[Nat])),
            List.cons(List.cons(Nat.1, List.cons(Nat.1, List.cons(Nat.1, List.nil[Nat]))), List.nil[List[Nat]]))
    List.cons(List.cons(Nat.3, List.nil[Nat]),
        List.cons(List.cons(Nat.2, List.cons(Nat.1, List.nil[Nat])),
            List.cons(List.cons(Nat.1, List.cons(Nat.1, List.cons(Nat.1, List.nil[Nat]))), List.nil[List[Nat]]))).filter(partition_set(Nat.3).contains) =
        List.cons(List.cons(Nat.3, List.nil[Nat]),
            List.cons(List.cons(Nat.2, List.cons(Nat.1, List.nil[Nat])),
                List.cons(List.cons(Nat.1, List.cons(Nat.1, List.cons(Nat.1, List.nil[Nat]))), List.nil[List[Nat]])))
    partitions_three.filter(partition_set(Nat.3).contains) = partitions_three
    List.cons(Nat.3, List.nil[Nat]) != List.cons(Nat.2, List.cons(Nat.1, List.nil[Nat]))
    List.cons(Nat.3, List.nil[Nat]) != List.cons(Nat.1, List.cons(Nat.1, List.cons(Nat.1, List.nil[Nat])))
    List.cons(Nat.2, List.cons(Nat.1, List.nil[Nat])) != List.cons(Nat.1, List.cons(Nat.1, List.cons(Nat.1, List.nil[Nat])))
    triple_is_unique(List.cons(Nat.3, List.nil[Nat]), List.cons(Nat.2, List.cons(Nat.1, List.nil[Nat])), List.cons(Nat.1, List.cons(Nat.1, List.cons(Nat.1, List.nil[Nat]))))
    List.cons(Nat.3, List.nil[Nat]) != List.cons(Nat.2, List.cons(Nat.1, List.nil[Nat])) and List.cons(Nat.3, List.nil[Nat]) != List.cons(Nat.1, List.cons(Nat.1, List.cons(Nat.1, List.nil[Nat]))) and List.cons(Nat.2, List.cons(Nat.1, List.nil[Nat])) != List.cons(Nat.1, List.cons(Nat.1, List.cons(Nat.1, List.nil[Nat]))) implies List.cons(List.cons(Nat.3, List.nil[Nat]), List.cons(List.cons(Nat.2, List.cons(Nat.1, List.nil[Nat])), List.cons(List.cons(Nat.1, List.cons(Nat.1, List.cons(Nat.1, List.nil[Nat]))), List.nil[List[Nat]]))).is_unique
    List.cons(List.cons(Nat.3, List.nil[Nat]), List.cons(List.cons(Nat.2, List.cons(Nat.1, List.nil[Nat])), List.cons(List.cons(Nat.1, List.cons(Nat.1, List.cons(Nat.1, List.nil[Nat]))), List.nil[List[Nat]]))).is_unique
    partitions_three.is_unique
    partitions_three.unique = partitions_three
    partitions_three.length = List.cons(List.cons(Nat.2, List.cons(Nat.1, List.nil[Nat])),
        List.cons(List.cons(Nat.1, List.cons(Nat.1, List.cons(Nat.1, List.nil[Nat]))), List.nil[List[Nat]])).length.suc
    List.cons(List.cons(Nat.2, List.cons(Nat.1, List.nil[Nat])),
        List.cons(List.cons(Nat.1, List.cons(Nat.1, List.cons(Nat.1, List.nil[Nat]))), List.nil[List[Nat]])).length = List.cons(List.cons(Nat.1, List.cons(Nat.1, List.cons(Nat.1, List.nil[Nat]))), List.nil[List[Nat]]).length.suc
    List.cons(List.cons(Nat.1, List.cons(Nat.1, List.cons(Nat.1, List.nil[Nat]))), List.nil[List[Nat]]).length = List.nil[List[Nat]].length.suc
    List.nil[List[Nat]].length = Nat.0
    List.cons(List.cons(Nat.1, List.cons(Nat.1, List.cons(Nat.1, List.nil[Nat]))), List.nil[List[Nat]]).length = Nat.0.suc
    List.cons(List.cons(Nat.2, List.cons(Nat.1, List.nil[Nat])),
        List.cons(List.cons(Nat.1, List.cons(Nat.1, List.cons(Nat.1, List.nil[Nat]))), List.nil[List[Nat]])).length = Nat.0.suc.suc
    partitions_three.length = Nat.0.suc.suc.suc
    Nat.0.suc.suc.suc = Nat.3
    partitions_three.length = Nat.3
    partitions_three.filter(partition_set(Nat.3).contains).unique = partitions_three
    partitions_three.filter(partition_set(Nat.3).contains).unique.length = Nat.3
    partitions_three.contains_set(partition_set(Nat.3)) and
        partitions_three.filter(partition_set(Nat.3).contains).unique.length = Nat.3
    exists(containing_list: List[List[Nat]]) {
        containing_list.contains_set(partition_set(Nat.3)) and
            containing_list.filter(partition_set(Nat.3).contains).unique.length = Nat.3
    }
    partition_set(Nat.3).cardinality_is(Nat.3)
    cardinality_is_well_defined(partition_set(Nat.3), p(Nat.3), Nat.3)
    partition_set(Nat.3).cardinality_is(p(Nat.3)) and partition_set(Nat.3).cardinality_is(Nat.3) implies p(Nat.3) = Nat.3
    p(Nat.3) = Nat.3
}

/// Prepending two elements to the empty list and appending a tail builds a
/// three-element cons list.
theorem add_cons_cons(h1: List[Nat], h2: List[Nat], rest: List[List[Nat]]) {
    List.cons(h1, List.cons(h2, List.nil[List[Nat]])) + rest = List.cons(h1, List.cons(h2, rest))
} by {
    List.cons(h1, List.cons(h2, List.nil[List[Nat]])) + rest =
        List.cons(h1, (List.cons(h2, List.nil[List[Nat]]) + rest))
    List.cons(h2, List.nil[List[Nat]]) + rest = List.cons(h2, (List.nil[List[Nat]] + rest))
    List.nil[List[Nat]] + rest = rest
    List.cons(h2, List.nil[List[Nat]]) + rest = List.cons(h2, rest)
    List.cons(h1, List.cons(h2, List.nil[List[Nat]])) + rest =
        List.cons(h1, List.cons(h2, rest))
}

/// Prepending three elements to the empty list and appending a tail builds a
/// four-element cons list.
theorem add_cons_cons_cons(h1: List[Nat], h2: List[Nat], h3: List[Nat], rest: List[List[Nat]]) {
    List.cons(h1, List.cons(h2, List.cons(h3, List.nil[List[Nat]]))) + rest =
        List.cons(h1, List.cons(h2, List.cons(h3, rest)))
} by {
    List.cons(h1, List.cons(h2, List.cons(h3, List.nil[List[Nat]]))) + rest =
        List.cons(h1, (List.cons(h2, List.cons(h3, List.nil[List[Nat]])) + rest))
    add_cons_cons(h2, h3, rest)
    List.cons(h2, List.cons(h3, List.nil[List[Nat]])) + rest = List.cons(h2, List.cons(h3, rest))
    List.cons(h1, List.cons(h2, List.cons(h3, List.nil[List[Nat]]))) + rest =
        List.cons(h1, List.cons(h2, List.cons(h3, rest)))
}

/// The parts of the partitions of four, as named constants.
let p4_a: List[Nat] = List.cons(Nat.4, List.nil[Nat])
let p4_b: List[Nat] = List.cons(Nat.3, List.cons(Nat.1, List.nil[Nat]))
let p4_c: List[Nat] = List.cons(Nat.2, List.cons(Nat.2, List.nil[Nat]))
let p4_d: List[Nat] = List.cons(Nat.2, List.cons(Nat.1, List.cons(Nat.1, List.nil[Nat])))
let p4_e: List[Nat] = List.cons(Nat.1, List.cons(Nat.1, List.cons(Nat.1, List.cons(Nat.1, List.nil[Nat]))))

/// Every partition of four occurs in the explicit list.
theorem partitions_four_contains(l: List[Nat]) {
    is_partition(l, Nat.4) implies partitions_four.contains(l)
} by {
    if is_partition(l, Nat.4) {
        partition_four_char(l)
        is_partition(l, Nat.4) = (l = p4_a or l = p4_b or l = p4_c or l = p4_d or l = p4_e)
        l = p4_a or l = p4_b or l = p4_c or l = p4_d or l = p4_e
        if l = p4_a {
            cons_contains_head(p4_a, List.cons(p4_b, List.cons(p4_c, List.cons(p4_d, List.cons(p4_e, List.nil[List[Nat]])))))
            List.cons(p4_a, List.cons(p4_b, List.cons(p4_c, List.cons(p4_d, List.cons(p4_e, List.nil[List[Nat]]))))).contains(p4_a)
            partitions_four.contains(p4_a)
            partitions_four.contains(l)
        }
        if not l = p4_a {
            if l = p4_b {
                cons_contains_head(p4_b, List.cons(p4_c, List.cons(p4_d, List.cons(p4_e, List.nil[List[Nat]]))))
                List.cons(p4_b, List.cons(p4_c, List.cons(p4_d, List.cons(p4_e, List.nil[List[Nat]])))).contains(p4_b)
                List.cons(p4_a, List.cons(p4_b, List.cons(p4_c, List.cons(p4_d, List.cons(p4_e, List.nil[List[Nat]]))))).contains(p4_b)
                partitions_four.contains(p4_b)
                partitions_four.contains(l)
            }
            if not l = p4_b {
                if l = p4_c {
                    cons_contains_head(p4_c, List.cons(p4_d, List.cons(p4_e, List.nil[List[Nat]])))
                    List.cons(p4_c, List.cons(p4_d, List.cons(p4_e, List.nil[List[Nat]]))).contains(p4_c)
                    List.cons(p4_b, List.cons(p4_c, List.cons(p4_d, List.cons(p4_e, List.nil[List[Nat]])))).contains(p4_c)
                    List.cons(p4_a, List.cons(p4_b, List.cons(p4_c, List.cons(p4_d, List.cons(p4_e, List.nil[List[Nat]]))))).contains(p4_c)
                    partitions_four.contains(p4_c)
                    partitions_four.contains(l)
                }
                if not l = p4_c {
                    if l = p4_d {
                        cons_contains_head(p4_d, List.cons(p4_e, List.nil[List[Nat]]))
                        List.cons(p4_d, List.cons(p4_e, List.nil[List[Nat]])).contains(p4_d)
                        List.cons(p4_c, List.cons(p4_d, List.cons(p4_e, List.nil[List[Nat]]))).contains(p4_d)
                        List.cons(p4_b, List.cons(p4_c, List.cons(p4_d, List.cons(p4_e, List.nil[List[Nat]])))).contains(p4_d)
                        List.cons(p4_a, List.cons(p4_b, List.cons(p4_c, List.cons(p4_d, List.cons(p4_e, List.nil[List[Nat]]))))).contains(p4_d)
                        partitions_four.contains(p4_d)
                        partitions_four.contains(l)
                    }
                    if not l = p4_d {
                        l = p4_e
                        cons_contains_head(p4_e, List.nil[List[Nat]])
                        List.cons(p4_e, List.nil[List[Nat]]).contains(p4_e)
                        List.cons(p4_d, List.cons(p4_e, List.nil[List[Nat]])).contains(p4_e)
                        List.cons(p4_c, List.cons(p4_d, List.cons(p4_e, List.nil[List[Nat]]))).contains(p4_e)
                        List.cons(p4_b, List.cons(p4_c, List.cons(p4_d, List.cons(p4_e, List.nil[List[Nat]])))).contains(p4_e)
                        List.cons(p4_a, List.cons(p4_b, List.cons(p4_c, List.cons(p4_d, List.cons(p4_e, List.nil[List[Nat]]))))).contains(p4_e)
                        partitions_four.contains(p4_e)
                        partitions_four.contains(l)
                    }
                    partitions_four.contains(l)
                }
                partitions_four.contains(l)
            }
            partitions_four.contains(l)
        }
        partitions_four.contains(l)
    }
}
/// Different heads give different cons lists.
theorem cons_neq_head(a: Nat, b: Nat, t1: List[Nat], t2: List[Nat]) {
    a != b implies List.cons(a, t1) != List.cons(b, t2)
} by {
    if a != b {
        if List.cons(a, t1) = List.cons(b, t2) {
            a = b
            false
        }
        List.cons(a, t1) != List.cons(b, t2)
    }
}

/// Equal heads with different second elements give different cons lists.
theorem cons_cons_neq_head(a: Nat, b: Nat, c: Nat, t1: List[Nat], t2: List[Nat]) {
    b != c implies List.cons(a, List.cons(b, t1)) != List.cons(a, List.cons(c, t2))
} by {
    if b != c {
        if List.cons(a, List.cons(b, t1)) = List.cons(a, List.cons(c, t2)) {
            List.cons(b, t1) = List.cons(c, t2)
            b = c
            false
        }
        List.cons(a, List.cons(b, t1)) != List.cons(a, List.cons(c, t2))
    }
}

/// The five parts of the partitions of four are pairwise distinct.
theorem p4_parts_distinct {
    p4_a != p4_b and p4_a != p4_c and p4_a != p4_d and p4_a != p4_e and
    p4_b != p4_c and p4_b != p4_d and p4_b != p4_e and
    p4_c != p4_d and p4_c != p4_e and p4_d != p4_e
} by {
    Nat.4 != Nat.3
    cons_neq_head(Nat.4, Nat.3, List.nil[Nat], List.cons(Nat.1, List.nil[Nat]))
    List.cons(Nat.4, List.nil[Nat]) != List.cons(Nat.3, List.cons(Nat.1, List.nil[Nat]))
    Nat.4 != Nat.2
    cons_neq_head(Nat.4, Nat.2, List.nil[Nat], List.cons(Nat.2, List.nil[Nat]))
    List.cons(Nat.4, List.nil[Nat]) != List.cons(Nat.2, List.cons(Nat.2, List.nil[Nat]))
    cons_neq_head(Nat.4, Nat.2, List.nil[Nat], List.cons(Nat.1, List.cons(Nat.1, List.nil[Nat])))
    List.cons(Nat.4, List.nil[Nat]) != List.cons(Nat.2, List.cons(Nat.1, List.cons(Nat.1, List.nil[Nat])))
    Nat.4 != Nat.1
    cons_neq_head(Nat.4, Nat.1, List.nil[Nat], List.cons(Nat.1, List.cons(Nat.1, List.cons(Nat.1, List.nil[Nat]))))
    List.cons(Nat.4, List.nil[Nat]) != List.cons(Nat.1, List.cons(Nat.1, List.cons(Nat.1, List.cons(Nat.1, List.nil[Nat]))))
    Nat.3 != Nat.2
    cons_neq_head(Nat.3, Nat.2, List.cons(Nat.1, List.nil[Nat]), List.cons(Nat.2, List.nil[Nat]))
    List.cons(Nat.3, List.cons(Nat.1, List.nil[Nat])) != List.cons(Nat.2, List.cons(Nat.2, List.nil[Nat]))
    cons_neq_head(Nat.3, Nat.2, List.cons(Nat.1, List.nil[Nat]), List.cons(Nat.1, List.cons(Nat.1, List.nil[Nat])))
    List.cons(Nat.3, List.cons(Nat.1, List.nil[Nat])) != List.cons(Nat.2, List.cons(Nat.1, List.cons(Nat.1, List.nil[Nat])))
    Nat.3 != Nat.1
    cons_neq_head(Nat.3, Nat.1, List.cons(Nat.1, List.nil[Nat]), List.cons(Nat.1, List.cons(Nat.1, List.cons(Nat.1, List.nil[Nat]))))
    List.cons(Nat.3, List.cons(Nat.1, List.nil[Nat])) != List.cons(Nat.1, List.cons(Nat.1, List.cons(Nat.1, List.cons(Nat.1, List.nil[Nat]))))
    Nat.2 != Nat.1
    cons_cons_neq_head(Nat.2, Nat.2, Nat.1, List.nil[Nat], List.cons(Nat.1, List.nil[Nat]))
    List.cons(Nat.2, List.cons(Nat.2, List.nil[Nat])) != List.cons(Nat.2, List.cons(Nat.1, List.cons(Nat.1, List.nil[Nat])))
    cons_neq_head(Nat.2, Nat.1, List.cons(Nat.2, List.nil[Nat]), List.cons(Nat.1, List.cons(Nat.1, List.cons(Nat.1, List.nil[Nat]))))
    List.cons(Nat.2, List.cons(Nat.2, List.nil[Nat])) != List.cons(Nat.1, List.cons(Nat.1, List.cons(Nat.1, List.cons(Nat.1, List.nil[Nat]))))
    cons_neq_head(Nat.2, Nat.1, List.cons(Nat.1, List.cons(Nat.1, List.nil[Nat])), List.cons(Nat.1, List.cons(Nat.1, List.cons(Nat.1, List.nil[Nat]))))
    List.cons(Nat.2, List.cons(Nat.1, List.cons(Nat.1, List.nil[Nat]))) != List.cons(Nat.1, List.cons(Nat.1, List.cons(Nat.1, List.cons(Nat.1, List.nil[Nat]))))
    p4_a = List.cons(Nat.4, List.nil[Nat])
    p4_b = List.cons(Nat.3, List.cons(Nat.1, List.nil[Nat]))
    p4_c = List.cons(Nat.2, List.cons(Nat.2, List.nil[Nat]))
    p4_d = List.cons(Nat.2, List.cons(Nat.1, List.cons(Nat.1, List.nil[Nat])))
    p4_e = List.cons(Nat.1, List.cons(Nat.1, List.cons(Nat.1, List.cons(Nat.1, List.nil[Nat]))))
    p4_a != p4_b
    p4_a != p4_c
    p4_a != p4_d
    p4_a != p4_e
    p4_b != p4_c
    p4_b != p4_d
    p4_b != p4_e
    p4_c != p4_d
    p4_c != p4_e
    p4_d != p4_e
    p4_a != p4_b and p4_a != p4_c and p4_a != p4_d and p4_a != p4_e and
        p4_b != p4_c and p4_b != p4_d and p4_b != p4_e and
        p4_c != p4_d and p4_c != p4_e and p4_d != p4_e
}

/// The explicit list of the partitions of four is duplicate-free.
theorem partitions_four_unique {
    partitions_four.is_unique
} by {
    quad_is_unique(p4_b, p4_c, p4_d, p4_e)
    p4_b != p4_c and p4_b != p4_d and p4_b != p4_e and p4_c != p4_d and p4_c != p4_e and p4_d != p4_e implies List.cons(p4_b, List.cons(p4_c, List.cons(p4_d, List.cons(p4_e, List.nil[List[Nat]])))).is_unique
    p4_parts_distinct
    p4_b != p4_c and p4_b != p4_d and p4_b != p4_e and p4_c != p4_d and p4_c != p4_e and p4_d != p4_e
    List.cons(p4_b, List.cons(p4_c, List.cons(p4_d, List.cons(p4_e, List.nil[List[Nat]])))).is_unique
    cons_contains_eq(p4_b, List.cons(p4_c, List.cons(p4_d, List.cons(p4_e, List.nil[List[Nat]]))), p4_a)
    List.cons(p4_b, List.cons(p4_c, List.cons(p4_d, List.cons(p4_e, List.nil[List[Nat]])))).contains(p4_a) =
        (p4_b = p4_a or List.cons(p4_c, List.cons(p4_d, List.cons(p4_e, List.nil[List[Nat]]))).contains(p4_a))
    cons_contains_eq(p4_c, List.cons(p4_d, List.cons(p4_e, List.nil[List[Nat]])), p4_a)
    List.cons(p4_c, List.cons(p4_d, List.cons(p4_e, List.nil[List[Nat]]))).contains(p4_a) =
        (p4_c = p4_a or List.cons(p4_d, List.cons(p4_e, List.nil[List[Nat]])).contains(p4_a))
    cons_contains_eq(p4_d, List.cons(p4_e, List.nil[List[Nat]]), p4_a)
    List.cons(p4_d, List.cons(p4_e, List.nil[List[Nat]])).contains(p4_a) =
        (p4_d = p4_a or List.cons(p4_e, List.nil[List[Nat]]).contains(p4_a))
    cons_contains_eq(p4_e, List.nil[List[Nat]], p4_a)
    List.cons(p4_e, List.nil[List[Nat]]).contains(p4_a) =
        (p4_e = p4_a or List.nil[List[Nat]].contains(p4_a))
    nil_not_contains(p4_a)
    not List.nil[List[Nat]].contains(p4_a)
    p4_parts_distinct
    p4_e != p4_a
    p4_d != p4_a
    p4_c != p4_a
    p4_b != p4_a
    not List.cons(p4_e, List.nil[List[Nat]]).contains(p4_a)
    not List.cons(p4_d, List.cons(p4_e, List.nil[List[Nat]])).contains(p4_a)
    not List.cons(p4_c, List.cons(p4_d, List.cons(p4_e, List.nil[List[Nat]]))).contains(p4_a)
    not List.cons(p4_b, List.cons(p4_c, List.cons(p4_d, List.cons(p4_e, List.nil[List[Nat]])))).contains(p4_a)
    cons_unique_of_tail_unique_not_contains(p4_a, List.cons(p4_b, List.cons(p4_c, List.cons(p4_d, List.cons(p4_e, List.nil[List[Nat]])))))
    List.cons(p4_b, List.cons(p4_c, List.cons(p4_d, List.cons(p4_e, List.nil[List[Nat]])))).is_unique and not List.cons(p4_b, List.cons(p4_c, List.cons(p4_d, List.cons(p4_e, List.nil[List[Nat]])))).contains(p4_a) implies List.cons(p4_a, List.cons(p4_b, List.cons(p4_c, List.cons(p4_d, List.cons(p4_e, List.nil[List[Nat]]))))).is_unique
    List.cons(p4_b, List.cons(p4_c, List.cons(p4_d, List.cons(p4_e, List.nil[List[Nat]])))).is_unique and not List.cons(p4_b, List.cons(p4_c, List.cons(p4_d, List.cons(p4_e, List.nil[List[Nat]])))).contains(p4_a)
    List.cons(p4_a, List.cons(p4_b, List.cons(p4_c, List.cons(p4_d, List.cons(p4_e, List.nil[List[Nat]]))))).is_unique
    partitions_four = List.cons(p4_a, List.cons(p4_b, List.cons(p4_c, List.cons(p4_d, List.cons(p4_e, List.nil[List[Nat]])))))
    partitions_four.is_unique
}

/// The or-form of the partitions of four implies being a partition of four.
theorem p4_sound(l: List[Nat]) {
    (l = p4_a or l = p4_b or l = p4_c or l = p4_d or l = p4_e) implies is_partition(l, Nat.4)
} by {
    if l = p4_a or l = p4_b or l = p4_c or l = p4_d or l = p4_e {
        p4_a = List.cons(Nat.4, List.nil[Nat])
        p4_b = List.cons(Nat.3, List.cons(Nat.1, List.nil[Nat]))
        p4_c = List.cons(Nat.2, List.cons(Nat.2, List.nil[Nat]))
        p4_d = List.cons(Nat.2, List.cons(Nat.1, List.cons(Nat.1, List.nil[Nat])))
        p4_e = List.cons(Nat.1, List.cons(Nat.1, List.cons(Nat.1, List.cons(Nat.1, List.nil[Nat]))))
        if l = p4_a {
            l = List.cons(Nat.4, List.nil[Nat])
            partition_four_sound(l)
            (l = List.cons(Nat.4, List.nil[Nat]) or l = List.cons(Nat.3, List.cons(Nat.1, List.nil[Nat])) or l = List.cons(Nat.2, List.cons(Nat.2, List.nil[Nat])) or l = List.cons(Nat.2, List.cons(Nat.1, List.cons(Nat.1, List.nil[Nat]))) or l = List.cons(Nat.1, List.cons(Nat.1, List.cons(Nat.1, List.cons(Nat.1, List.nil[Nat]))))) implies is_partition(l, Nat.4)
            l = List.cons(Nat.4, List.nil[Nat]) or l = List.cons(Nat.3, List.cons(Nat.1, List.nil[Nat])) or l = List.cons(Nat.2, List.cons(Nat.2, List.nil[Nat])) or l = List.cons(Nat.2, List.cons(Nat.1, List.cons(Nat.1, List.nil[Nat]))) or l = List.cons(Nat.1, List.cons(Nat.1, List.cons(Nat.1, List.cons(Nat.1, List.nil[Nat]))))
            is_partition(l, Nat.4)
        }
        if not l = p4_a {
            if l = p4_b {
                l = List.cons(Nat.3, List.cons(Nat.1, List.nil[Nat]))
                partition_four_sound(l)
                (l = List.cons(Nat.4, List.nil[Nat]) or l = List.cons(Nat.3, List.cons(Nat.1, List.nil[Nat])) or l = List.cons(Nat.2, List.cons(Nat.2, List.nil[Nat])) or l = List.cons(Nat.2, List.cons(Nat.1, List.cons(Nat.1, List.nil[Nat]))) or l = List.cons(Nat.1, List.cons(Nat.1, List.cons(Nat.1, List.cons(Nat.1, List.nil[Nat]))))) implies is_partition(l, Nat.4)
                l = List.cons(Nat.4, List.nil[Nat]) or l = List.cons(Nat.3, List.cons(Nat.1, List.nil[Nat])) or l = List.cons(Nat.2, List.cons(Nat.2, List.nil[Nat])) or l = List.cons(Nat.2, List.cons(Nat.1, List.cons(Nat.1, List.nil[Nat]))) or l = List.cons(Nat.1, List.cons(Nat.1, List.cons(Nat.1, List.cons(Nat.1, List.nil[Nat]))))
                is_partition(l, Nat.4)
            }
            if not l = p4_b {
                if l = p4_c {
                    l = List.cons(Nat.2, List.cons(Nat.2, List.nil[Nat]))
                    partition_four_sound(l)
                    (l = List.cons(Nat.4, List.nil[Nat]) or l = List.cons(Nat.3, List.cons(Nat.1, List.nil[Nat])) or l = List.cons(Nat.2, List.cons(Nat.2, List.nil[Nat])) or l = List.cons(Nat.2, List.cons(Nat.1, List.cons(Nat.1, List.nil[Nat]))) or l = List.cons(Nat.1, List.cons(Nat.1, List.cons(Nat.1, List.cons(Nat.1, List.nil[Nat]))))) implies is_partition(l, Nat.4)
                    l = List.cons(Nat.4, List.nil[Nat]) or l = List.cons(Nat.3, List.cons(Nat.1, List.nil[Nat])) or l = List.cons(Nat.2, List.cons(Nat.2, List.nil[Nat])) or l = List.cons(Nat.2, List.cons(Nat.1, List.cons(Nat.1, List.nil[Nat]))) or l = List.cons(Nat.1, List.cons(Nat.1, List.cons(Nat.1, List.cons(Nat.1, List.nil[Nat]))))
                    is_partition(l, Nat.4)
                }
                if not l = p4_c {
                    if l = p4_d {
                        l = List.cons(Nat.2, List.cons(Nat.1, List.cons(Nat.1, List.nil[Nat])))
                        partition_four_sound(l)
                        (l = List.cons(Nat.4, List.nil[Nat]) or l = List.cons(Nat.3, List.cons(Nat.1, List.nil[Nat])) or l = List.cons(Nat.2, List.cons(Nat.2, List.nil[Nat])) or l = List.cons(Nat.2, List.cons(Nat.1, List.cons(Nat.1, List.nil[Nat]))) or l = List.cons(Nat.1, List.cons(Nat.1, List.cons(Nat.1, List.cons(Nat.1, List.nil[Nat]))))) implies is_partition(l, Nat.4)
                        l = List.cons(Nat.4, List.nil[Nat]) or l = List.cons(Nat.3, List.cons(Nat.1, List.nil[Nat])) or l = List.cons(Nat.2, List.cons(Nat.2, List.nil[Nat])) or l = List.cons(Nat.2, List.cons(Nat.1, List.cons(Nat.1, List.nil[Nat]))) or l = List.cons(Nat.1, List.cons(Nat.1, List.cons(Nat.1, List.cons(Nat.1, List.nil[Nat]))))
                        is_partition(l, Nat.4)
                    }
                    if not l = p4_d {
                        l = p4_e
                        l = List.cons(Nat.1, List.cons(Nat.1, List.cons(Nat.1, List.cons(Nat.1, List.nil[Nat]))))
                        partition_four_sound(l)
                        (l = List.cons(Nat.4, List.nil[Nat]) or l = List.cons(Nat.3, List.cons(Nat.1, List.nil[Nat])) or l = List.cons(Nat.2, List.cons(Nat.2, List.nil[Nat])) or l = List.cons(Nat.2, List.cons(Nat.1, List.cons(Nat.1, List.nil[Nat]))) or l = List.cons(Nat.1, List.cons(Nat.1, List.cons(Nat.1, List.cons(Nat.1, List.nil[Nat])))) )implies is_partition(l, Nat.4)
                        l = List.cons(Nat.4, List.nil[Nat]) or l = List.cons(Nat.3, List.cons(Nat.1, List.nil[Nat])) or l = List.cons(Nat.2, List.cons(Nat.2, List.nil[Nat])) or l = List.cons(Nat.2, List.cons(Nat.1, List.cons(Nat.1, List.nil[Nat]))) or l = List.cons(Nat.1, List.cons(Nat.1, List.cons(Nat.1, List.cons(Nat.1, List.nil[Nat]))))
                        is_partition(l, Nat.4)
                    }
                    is_partition(l, Nat.4)
                }
                is_partition(l, Nat.4)
            }
            is_partition(l, Nat.4)
        }
        is_partition(l, Nat.4)
    }
}

/// Membership in the explicit list of partitions of four means being a
/// partition of four.
theorem partitions_four_membership(l: List[Nat]) {
    partitions_four.contains(l) implies is_partition(l, Nat.4)
} by {
    if partitions_four.contains(l) {
        cons_contains_eq(p4_a, List.cons(p4_b, List.cons(p4_c, List.cons(p4_d, List.cons(p4_e, List.nil[List[Nat]])))), l)
        partitions_four.contains(l) =
            (l = p4_a or List.cons(p4_b, List.cons(p4_c, List.cons(p4_d, List.cons(p4_e, List.nil[List[Nat]])))).contains(l))
        cons_contains_eq(p4_b, List.cons(p4_c, List.cons(p4_d, List.cons(p4_e, List.nil[List[Nat]]))), l)
        List.cons(p4_b, List.cons(p4_c, List.cons(p4_d, List.cons(p4_e, List.nil[List[Nat]])))).contains(l) =
            (l = p4_b or List.cons(p4_c, List.cons(p4_d, List.cons(p4_e, List.nil[List[Nat]]))).contains(l))
        cons_contains_eq(p4_c, List.cons(p4_d, List.cons(p4_e, List.nil[List[Nat]])), l)
        List.cons(p4_c, List.cons(p4_d, List.cons(p4_e, List.nil[List[Nat]]))).contains(l) =
            (l = p4_c or List.cons(p4_d, List.cons(p4_e, List.nil[List[Nat]])).contains(l))
        cons_contains_eq(p4_d, List.cons(p4_e, List.nil[List[Nat]]), l)
        List.cons(p4_d, List.cons(p4_e, List.nil[List[Nat]])).contains(l) =
            (l = p4_d or List.cons(p4_e, List.nil[List[Nat]]).contains(l))
        cons_contains_eq(p4_e, List.nil[List[Nat]], l)
        List.cons(p4_e, List.nil[List[Nat]]).contains(l) =
            (l = p4_e or List.nil[List[Nat]].contains(l))
        nil_not_contains(l)
        l = p4_a or l = p4_b or l = p4_c or l = p4_d or l = p4_e
        p4_sound(l)
        (l = p4_a or l = p4_b or l = p4_c or l = p4_d or l = p4_e) implies is_partition(l, Nat.4)
        is_partition(l, Nat.4)
    }
}

/// Filtering a list with a predicate that holds on every element keeps the
/// whole list.
theorem filter_all_keep[T](l: List[T], f: T -> Bool) {
    (forall(x: T) { l.contains(x) implies f(x) }) implies l.filter(f) = l
} by {
    define q(xs: List[T]) -> Bool {
        (forall(x: T) { xs.contains(x) implies f(x) }) implies xs.filter(f) = xs
    }
    if forall(x: T) { List.nil[T].contains(x) implies f(x) } {
        List.nil[T].filter(f) = List.nil[T]
    }
    q(List.nil[T])
    forall(head: T, tail: List[T]) {
        if q(tail) {
            if forall(x: T) { List.cons(head, tail).contains(x) implies f(x) } {
                f(head)
                forall(x: T) {
                    if tail.contains(x) {
                        List.cons(head, tail).contains(x)
                        f(x)
                    }
                }
                forall(x: T) { tail.contains(x) implies f(x) }
                q(tail)
                (forall(x: T) { tail.contains(x) implies f(x) }) implies tail.filter(f) = tail
                tail.filter(f) = tail
                List.cons(head, tail).filter(f) = List.cons(head, tail.filter(f))
                List.cons(head, tail.filter(f)) = List.cons(head, tail)
                List.cons(head, tail).filter(f) = List.cons(head, tail)
            }
            q(List.cons(head, tail))
        }
    }
    List.induction(function(xs: List[T]) { q(xs) })
    q(l)
}

/// The partitions of four are 4, 3 + 1, 2 + 2, 2 + 1 + 1 and 1 + 1 + 1 + 1,
/// so p(4) = 5.
theorem p_four {
    p(Nat.4) = Nat.5
} by {
    partition_set(Nat.4).cardinality_is(p(Nat.4))
    contains_set_intro(partitions_four, partition_set(Nat.4))
    forall(l: List[Nat]) {
        if partition_set(Nat.4).contains(l) {
            partition_set_contains_iff(Nat.4, l)
            partition_set(Nat.4).contains(l) = is_partition(l, Nat.4)
            is_partition(l, Nat.4)
            partitions_four_contains(l)
            is_partition(l, Nat.4) implies partitions_four.contains(l)
            partitions_four.contains(l)
        }
    }
    forall(l: List[Nat]) { partition_set(Nat.4).contains(l) implies partitions_four.contains(l) }
    partitions_four.contains_set(partition_set(Nat.4))
    partition_four_char(p4_a)
    is_partition(p4_a, Nat.4) = (p4_a = p4_a or p4_a = p4_b or p4_a = p4_c or p4_a = p4_d or p4_a = p4_e)
    p4_a = p4_a
    is_partition(p4_a, Nat.4) = true
    partition_set_contains_iff(Nat.4, p4_a)
    partition_set(Nat.4).contains(p4_a) = is_partition(p4_a, Nat.4)
    partition_set(Nat.4).contains(p4_a) = true
    partition_set(Nat.4).contains(p4_a)
    partition_four_char(p4_b)
    is_partition(p4_b, Nat.4) = (p4_b = p4_a or p4_b = p4_b or p4_b = p4_c or p4_b = p4_d or p4_b = p4_e)
    p4_b = p4_b
    is_partition(p4_b, Nat.4) = true
    partition_set_contains_iff(Nat.4, p4_b)
    partition_set(Nat.4).contains(p4_b) = true
    partition_set(Nat.4).contains(p4_b)
    partition_four_char(p4_c)
    is_partition(p4_c, Nat.4) = (p4_c = p4_a or p4_c = p4_b or p4_c = p4_c or p4_c = p4_d or p4_c = p4_e)
    p4_c = p4_c
    is_partition(p4_c, Nat.4) = true
    partition_set_contains_iff(Nat.4, p4_c)
    partition_set(Nat.4).contains(p4_c) = true
    partition_set(Nat.4).contains(p4_c)
    partition_four_char(p4_d)
    is_partition(p4_d, Nat.4) = (p4_d = p4_a or p4_d = p4_b or p4_d = p4_c or p4_d = p4_d or p4_d = p4_e)
    p4_d = p4_d
    is_partition(p4_d, Nat.4) = true
    partition_set_contains_iff(Nat.4, p4_d)
    partition_set(Nat.4).contains(p4_d) = true
    partition_set(Nat.4).contains(p4_d)
    partition_four_char(p4_e)
    is_partition(p4_e, Nat.4) = (p4_e = p4_a or p4_e = p4_b or p4_e = p4_c or p4_e = p4_d or p4_e = p4_e)
    p4_e = p4_e
    is_partition(p4_e, Nat.4) = true
    partition_set_contains_iff(Nat.4, p4_e)
    partition_set(Nat.4).contains(p4_e) = true
    partition_set(Nat.4).contains(p4_e)
    forall(x: List[Nat]) {
        if partitions_four.contains(x) {
            partitions_four_membership(x)
            partitions_four.contains(x) implies is_partition(x, Nat.4)
            is_partition(x, Nat.4)
            partition_set_contains_iff(Nat.4, x)
            partition_set(Nat.4).contains(x) = is_partition(x, Nat.4)
            partition_set(Nat.4).contains(x)
        }
    }
    forall(x: List[Nat]) { partitions_four.contains(x) implies partition_set(Nat.4).contains(x) }
    filter_all_keep(partitions_four, partition_set(Nat.4).contains)
    (forall(x: List[Nat]) { partitions_four.contains(x) implies partition_set(Nat.4).contains(x) }) implies partitions_four.filter(partition_set(Nat.4).contains) = partitions_four
    partitions_four.filter(partition_set(Nat.4).contains) = partitions_four
    partitions_four_unique
    partitions_four.is_unique
    partitions_four.unique = partitions_four
    partitions_four.length = List.cons(p4_b, List.cons(p4_c, List.cons(p4_d, List.cons(p4_e, List.nil[List[Nat]])))).length.suc
    List.cons(p4_b, List.cons(p4_c, List.cons(p4_d, List.cons(p4_e, List.nil[List[Nat]])))).length = List.cons(p4_c, List.cons(p4_d, List.cons(p4_e, List.nil[List[Nat]]))).length.suc
    List.cons(p4_c, List.cons(p4_d, List.cons(p4_e, List.nil[List[Nat]]))).length = List.cons(p4_d, List.cons(p4_e, List.nil[List[Nat]])).length.suc
    List.cons(p4_d, List.cons(p4_e, List.nil[List[Nat]])).length = List.cons(p4_e, List.nil[List[Nat]]).length.suc
    List.cons(p4_e, List.nil[List[Nat]]).length = List.nil[List[Nat]].length.suc
    List.nil[List[Nat]].length = Nat.0
    List.cons(p4_e, List.nil[List[Nat]]).length = Nat.0.suc
    List.cons(p4_d, List.cons(p4_e, List.nil[List[Nat]])).length = Nat.0.suc.suc
    List.cons(p4_c, List.cons(p4_d, List.cons(p4_e, List.nil[List[Nat]]))).length = Nat.0.suc.suc.suc
    List.cons(p4_b, List.cons(p4_c, List.cons(p4_d, List.cons(p4_e, List.nil[List[Nat]])))).length = Nat.0.suc.suc.suc.suc
    partitions_four.length = Nat.0.suc.suc.suc.suc.suc
    Nat.0.suc.suc.suc.suc.suc = Nat.5
    partitions_four.length = Nat.5
    partitions_four.filter(partition_set(Nat.4).contains).unique = partitions_four
    partitions_four.filter(partition_set(Nat.4).contains).unique.length = Nat.5
    partitions_four.contains_set(partition_set(Nat.4)) and
        partitions_four.filter(partition_set(Nat.4).contains).unique.length = Nat.5
    exists(containing_list: List[List[Nat]]) {
        containing_list.contains_set(partition_set(Nat.4)) and
            containing_list.filter(partition_set(Nat.4).contains).unique.length = Nat.5
    }
    partition_set(Nat.4).cardinality_is(Nat.5)
    cardinality_is_well_defined(partition_set(Nat.4), p(Nat.4), Nat.5)
    partition_set(Nat.4).cardinality_is(p(Nat.4)) and partition_set(Nat.4).cardinality_is(Nat.5) implies p(Nat.4) = Nat.5
    p(Nat.4) = Nat.5
}

// ---------------------------------------------------------------------------
// The classic identities.
// ---------------------------------------------------------------------------
//
// The generating function identity.  For |x| < 1,
//
//     Π_{k >= 1} 1 / (1 - x^k)  =  Σ_{n >= 0} p(n) x^n,
//
// where the left-hand side is the infinite product over the geometric series
// 1 + x^k + x^(2k) + ...: expanding the product, the coefficient of x^n
// counts, for each part size k, how many copies of x^k multiply together,
// which is exactly the number of ways to write n as a sum of positive
// integers.  Proving it in this library needs the infinite-product machinery
// (the sequence of partial products Π_{k <= N} 1/(1-x^k) and its limit, in
// the style of `generating_functions.ac`) together with the identity
//
//     p(n) = |{ parts : List[Nat] | is_partition(parts, n) }|,
//
// which is the set-cardinality form of `p` used above.
//
// Euler's pentagonal number theorem.  For |x| < 1,
//
//     Π_{k >= 1} (1 - x^k)  =  Σ_{m in Z} (-1)^m x^(m(3m-1)/2)
//                            =  1 - x - x^2 + x^5 + x^7 - x^12 - x^15 + ...,
//
// with the exponents m(3m-1)/2 the generalized pentagonal numbers.  Its
// coefficient form is the recurrence
//
//     p(n) = p(n-1) + p(n-2) - p(n-5) - p(n-7) + p(n-12) + p(n-15) - ...,
//
// which also yields the partition function efficiently.  Both identities are
// famous and non-trivial; they are recorded here as statements for later
// work, together with a table of the values proved above.
//
//     n :  0  1  2  3  4
//     p :  1  1  2  3  5
