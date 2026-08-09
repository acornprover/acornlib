/// The Erdős–Ginzburg–Ziv theorem (Freek Top-100 #37): among any `2n - 1`
/// integers there are `n` whose sum is divisible by `n`.
///
/// The proof is by the classical small-case analysis: `n = 1` is trivial, `n = 2`
/// is a pigeonhole argument on parity, and `n = 3` is the classic pigeonhole
/// argument on residues modulo three (either three of the five numbers share a
/// residue, or all three residue classes occur, and one of each sums to zero).
///
/// The general theorem for every `n` requires the Chevalley–Warning theorem
/// (or the zero-sum induction over prime divisors of `n`) and is not
/// formalized here; see the comment at the end of this file.
///
/// The sequence is presented as a function `f: Nat -> Nat`, and a subsequence
/// of `n` elements is presented by its positions (an index map in the general
/// statement below).  The index-map machinery and the divisibility arithmetic
/// live in the companion modules `erdos_ginzburg_ziv_indices` and
/// `erdos_ginzburg_ziv_div`.

from nat import Nat, not_lt_zero, lt_suc_right, lt_and_lte, lte_and_lt, lt_suc,
    lt_trans, lte_imp_not_lt, lt_not_ref, mul_one_left, mul_one_right, divides_self,
    divides_mul, mul_suc_right, mul_comm, div_imp_mod, mod_of_zero, div_sub_mod,
    sub_zero, lt_cancel_suc
from list import List, map, sum, partial
from combinatorics.erdos_ginzburg_ziv_indices import pick_one, pick_two,
    egz_bounds_one, egz_inj_one, lt_two_cases, lt_two_ne_zero_imp_one,
    lt_two_ne_one_imp_zero
from combinatorics.erdos_ginzburg_ziv_div import one_divides, even_sum_of_same_parity,
    mod_two_lt, three_divides_of_equal_residues, three_divides_of_all_residues

numerals Nat

// === The case n = 1 ===

/// The Erdős–Ginzburg–Ziv theorem for `n = 1`: among the one integer below one
/// there is one (namely it) whose sum is divisible by one.
theorem egz_n1(f: Nat -> Nat) {
    exists(i: Nat) {
        i < Nat.1 and Nat.1.divides(f(i))
    }
} by {
    lt_suc(Nat.0)
    Nat.0 < Nat.1
    one_divides(f(Nat.0))
    exists(i: Nat) {
        i < Nat.1 and Nat.1.divides(f(i))
    }
}

// === The case n = 2 ===

/// The Erdős–Ginzburg–Ziv theorem for `n = 2`: among three integers, two have
/// an even sum, by pigeonhole on parity.
///
/// The proof splits on the parities of the three values: if the first two share
/// a parity, they are the pair; otherwise, either the first and the third share
/// a parity, or the second and the third do (two distinct values among the two
/// parities force the remaining two to agree).
theorem egz_n2(f: Nat -> Nat) {
    exists(i: Nat, j: Nat) {
        i < j and j < Nat.3 and Nat.2.divides(f(i) + f(j))
    }
} by {
    if f(0).mod(2) = f(1).mod(2) {
        even_sum_of_same_parity(f(0), f(1))
        Nat.2.divides(f(0) + f(1))
        lt_suc(Nat.0)
        Nat.0 < Nat.1
        lt_suc(Nat.1)
        Nat.1 < Nat.2
        lt_suc(Nat.2)
        Nat.2 < Nat.3
        lt_trans(Nat.1, Nat.2, Nat.3)
        Nat.1 < Nat.3
        exists(i: Nat, j: Nat) {
            i < j and j < Nat.3 and Nat.2.divides(f(i) + f(j))
        }
    }
    if f(0).mod(2) != f(1).mod(2) {
        if f(0).mod(2) = f(2).mod(2) {
            even_sum_of_same_parity(f(0), f(2))
            Nat.2.divides(f(0) + f(2))
            lt_suc(Nat.0)
            Nat.0 < Nat.1
            lt_suc(Nat.2)
            Nat.2 < Nat.3
            lt_trans(Nat.1, Nat.2, Nat.3)
            Nat.1 < Nat.3
            lt_trans(Nat.0, Nat.1, Nat.3)
            Nat.0 < Nat.3
            exists(i: Nat, j: Nat) {
                i < j and j < Nat.3 and Nat.2.divides(f(i) + f(j))
            }
        }
        if f(0).mod(2) != f(2).mod(2) {
            mod_two_lt(f(0))
            mod_two_lt(f(1))
            mod_two_lt(f(2))
            lt_two_cases(f(0).mod(2))
            f(0).mod(2) = Nat.0 or f(0).mod(2) = Nat.1
            if f(0).mod(2) = Nat.0 {
                f(1).mod(2) != f(0).mod(2)
                f(2).mod(2) != f(0).mod(2)
                lt_two_ne_zero_imp_one(f(1).mod(2))
                f(1).mod(2) = Nat.1
                lt_two_ne_zero_imp_one(f(2).mod(2))
                f(2).mod(2) = Nat.1
                f(1).mod(2) = f(2).mod(2)
            }
            if f(0).mod(2) = Nat.1 {
                f(1).mod(2) != f(0).mod(2)
                f(2).mod(2) != f(0).mod(2)
                lt_two_ne_one_imp_zero(f(1).mod(2))
                f(1).mod(2) = Nat.0
                lt_two_ne_one_imp_zero(f(2).mod(2))
                f(2).mod(2) = Nat.0
                f(1).mod(2) = f(2).mod(2)
            }
            f(1).mod(2) = f(2).mod(2)
            even_sum_of_same_parity(f(1), f(2))
            Nat.2.divides(f(1) + f(2))
            lt_suc(Nat.1)
            Nat.1 < Nat.2
            lt_suc(Nat.2)
            Nat.2 < Nat.3
            exists(i: Nat, j: Nat) {
                i < j and j < Nat.3 and Nat.2.divides(f(i) + f(j))
            }
        }
        exists(i: Nat, j: Nat) {
            i < j and j < Nat.3 and Nat.2.divides(f(i) + f(j))
        }
    }
    exists(i: Nat, j: Nat) {
        i < j and j < Nat.3 and Nat.2.divides(f(i) + f(j))
    }
}

// === The case n = 3 ===

/// Three positions with equal residues modulo three: their values sum to a
/// multiple of three.
theorem egz_n3_equal(f: Nat -> Nat, a: Nat, b: Nat, c: Nat) {
    a < b and b < c and c < Nat.5 and
    f(a).mod(3) = f(b).mod(3) and f(b).mod(3) = f(c).mod(3) implies
    Nat.3.divides(f(a) + f(b) + f(c))
} by {
    if a < b and b < c and c < Nat.5 and
        f(a).mod(3) = f(b).mod(3) and f(b).mod(3) = f(c).mod(3) {
        three_divides_of_equal_residues(f, a, b, c, f(a).mod(3))
        f(a).mod(3) = f(a).mod(3) and f(b).mod(3) = f(a).mod(3) and f(c).mod(3) = f(a).mod(3)
        Nat.3.divides(f(a) + f(b) + f(c))
    }
}

/// Three positions with residues zero, one and two: their values sum to a
/// multiple of three.
theorem egz_n3_all(f: Nat -> Nat, a: Nat, b: Nat, c: Nat) {
    a < Nat.5 and b < Nat.5 and c < Nat.5 and
    f(a).mod(3) = Nat.0 and f(b).mod(3) = Nat.1 and f(c).mod(3) = Nat.2 implies
    Nat.3.divides(f(a) + f(b) + f(c))
} by {
    if a < Nat.5 and b < Nat.5 and c < Nat.5 and
        f(a).mod(3) = Nat.0 and f(b).mod(3) = Nat.1 and f(c).mod(3) = Nat.2 {
        three_divides_of_all_residues(f, a, b, c)
        f(a).mod(3) = Nat.0 and f(b).mod(3) = Nat.1 and f(c).mod(3) = Nat.2
        Nat.3.divides(f(a) + f(b) + f(c))
    }
}

/// The case `n = 3` in full: among the five values of `f`, three have a sum
/// divisible by three.
///
/// The classical argument: the residues of the five values modulo three are
/// counted (`erdos_ginzburg_ziv_count`).  Either some residue class contains
/// three of the values, in which case `egz_n3_equal` applies, or every class
/// occurs at least once, in which case one value of each residue is chosen
/// and `egz_n3_all` applies.  The counting dichotomy is recorded in the
/// companion module but the extraction of the three positions from the counts
/// is not yet formalized.
///
/// General statement of the Erdős–Ginzburg–Ziv theorem (unproved here; the
/// general case needs the Chevalley–Warning theorem or the zero-sum induction
/// over the prime divisors of `n`):
///
///   forall(n: Nat, f: Nat -> Nat) {
///       Nat.1 <= n implies exists(w: Nat -> Nat) {
///           (forall(j: Nat) { j < n implies w(j) < 2 * n - 1 }) and
///           ((forall(a: Nat, b: Nat) { a < b and b < n implies w(a) != w(b) }) and
///            n.divides(partial(function(j: Nat) { f(w(j)) }, n)))
///       }
///   }
