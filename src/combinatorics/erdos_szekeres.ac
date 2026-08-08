/// The Erdős–Szekeres theorem (Freek Top-100 #73): any sequence of more than `r * s`
/// distinct elements of a linear order contains an increasing subsequence of length
/// `r + 1` or a decreasing subsequence of length `s + 1`.
///
/// The proof assigns to each position `i` the pair `(a_i, b_i)` where `a_i` is the
/// length of the longest increasing subsequence starting at `i` and `b_i` is the
/// length of the longest decreasing subsequence starting at `i`.  These pairs are
/// pairwise distinct (if `x_i < x_j` then `a_i >= a_j + 1`, and if `x_j < x_i` then
/// `b_i >= b_j + 1`), and under the assumption that no increasing subsequence of
/// length `r + 1` and no decreasing subsequence of length `s + 1` exists every pair
/// lies in the box `{1, ..., r} x {1, ..., s}`, which has only `r * s` elements —
/// contradicting the pigeonhole principle.

from nat import Nat, not_lt_zero, lt_suc_right, zero_or_suc, trichotomy, lt_cancel_suc,
    lt_and_lte, lte_and_lt, lt_imp_lte_suc, lte_cancel_suc, lt_suc, suc_sub_one, add_sub,
    sub_zero, lte_add_left, mul_comm, lte_imp_not_lt, only_zero_lte_zero, lte_trans,
    lt_add_suc, add_zero_left, lt_or_lte, lt_not_ref, mul_suc_right
from list import List, map, add_length, add_contains_right, add_contains_left,
    map_contains, map_length, map_contains_of_contains, range_contains_of_lt,
    lt_of_range_contains, range_is_unique, length_range,
    unique_is_smallest_containing_list, unique_implies_no_duplicate,
    list_contains_implies_count_geq_one, unique_implies_tail_unique
from data.list.list_pigeonhole import pigeonhole_unique_map_in, locally_injective_map_is_unique
from data.nat.nat_bounded_max import has_max, is_max, is_max_apply, is_max_is_upper_bound,
    is_upper_bound_of, is_upper_bound_of_intro
from order import LinearOrder, lt_trans, lt_imp_ne, lte_or_lte_swap
from pair import Pair, pair_new_first, pair_new_second, pair_ext

numerals Nat

/// The element at position `i` of the list, if `i` is in bounds.
define value_at[T](xs: List[T], i: Nat, x: T) -> Bool {
    xs.get_idx(i) = Option.some(x)
}

/// The value at position `i` is strictly less than the value at position `j`.
define lt_at[T: LinearOrder](xs: List[T], i: Nat, j: Nat) -> Bool {
    exists(x: T, y: T) {
        value_at(xs, i, x) and value_at(xs, j, y) and x < y
    }
}

/// The value at position `i` is strictly greater than the value at position `j`.
define gt_at[T: LinearOrder](xs: List[T], i: Nat, j: Nat) -> Bool {
    lt_at(xs, j, i)
}

/// A witness chain of positions of length `k` whose values are `cmp`-comparable.
///
/// `w(0), ..., w(k - 1)` are strictly increasing in-bounds positions and
/// `cmp(w(a), w(b))` holds for every `a < b < k`.
define mono_chain[T: LinearOrder](xs: List[T], w: Nat -> Nat, cmp: Nat -> Nat -> Bool, k: Nat) -> Bool {
    forall(a: Nat, b: Nat) {
        a < b and b < k implies
            w(a) < w(b) and w(b) < xs.length and cmp(w(a), w(b))
    }
}

/// There is a chain of length at least `k` starting at position `i`.
define chain_geq_from[T: LinearOrder](xs: List[T], cmp: Nat -> Nat -> Bool, i: Nat, k: Nat) -> Bool {
    exists(w: Nat -> Nat) {
        w(0) = i and mono_chain(xs, w, cmp, k)
    }
}

/// The list contains a chain of length at least `k`.
define chain_geq[T: LinearOrder](xs: List[T], cmp: Nat -> Nat -> Bool, k: Nat) -> Bool {
    exists(w: Nat -> Nat) {
        mono_chain(xs, w, cmp, k)
    }
}

/// The empty chain is a chain.
theorem mono_chain_zero[T: LinearOrder](xs: List[T], w: Nat -> Nat, cmp: Nat -> Nat -> Bool) {
    mono_chain(xs, w, cmp, Nat.0)
} by {
    forall(a: Nat, b: Nat) {
        if a < b and b < Nat.0 {
            not_lt_zero(b)
            false
        }
    }
}

/// A singleton chain is a chain.
theorem mono_chain_one[T: LinearOrder](xs: List[T], w: Nat -> Nat, cmp: Nat -> Nat -> Bool) {
    mono_chain(xs, w, cmp, Nat.1)
} by {
    forall(a: Nat, b: Nat) {
        if a < b and b < Nat.1 {
            lt_suc_right(b, Nat.0)
            b = Nat.0 or b < Nat.0
            if b = Nat.0 {
                not_lt_zero(a)
                a < Nat.0
                false
            }
            if b < Nat.0 {
                not_lt_zero(b)
                false
            }
            false
        }
    }
}

/// A chain and a pair of in-bounds indices yield the chain constraint at that pair.
theorem mono_chain_apply[T: LinearOrder](xs: List[T], w: Nat -> Nat, cmp: Nat -> Nat -> Bool, k: Nat, a: Nat, b: Nat) {
    mono_chain(xs, w, cmp, k) and a < b and b < k implies
    w(a) < w(b) and w(b) < xs.length and cmp(w(a), w(b))
} by {
    if mono_chain(xs, w, cmp, k) and a < b and b < k {
        w(a) < w(b) and w(b) < xs.length and cmp(w(a), w(b))
    }
}

/// A chain of length `m` restricts to a chain of length `k <= m`.
theorem mono_chain_mono[T: LinearOrder](xs: List[T], w: Nat -> Nat, cmp: Nat -> Nat -> Bool, k: Nat, m: Nat) {
    mono_chain(xs, w, cmp, m) and k <= m implies mono_chain(xs, w, cmp, k)
} by {
    if mono_chain(xs, w, cmp, m) and k <= m {
        forall(a: Nat, b: Nat) {
            if a < b and b < k {
                lt_and_lte(b, k, m)
                b < m
                mono_chain_apply(xs, w, cmp, m, a, b)
                a < b and b < m
                w(a) < w(b) and w(b) < xs.length and cmp(w(a), w(b))
            }
        }
    }
}

/// There is a chain of length at least zero starting at position `i`.
theorem chain_geq_from_zero[T: LinearOrder](xs: List[T], cmp: Nat -> Nat -> Bool, i: Nat) {
    chain_geq_from(xs, cmp, i, Nat.0)
}

/// There is a chain of length at least one starting at an in-bounds position.
theorem chain_geq_from_one[T: LinearOrder](xs: List[T], cmp: Nat -> Nat -> Bool, i: Nat) {
    i < xs.length implies chain_geq_from(xs, cmp, i, Nat.1)
} by {
    if i < xs.length {
        mono_chain_one(xs, constant[Nat, Nat](i), cmp)
        constant[Nat, Nat](i)(0) = i
        exists(w: Nat -> Nat) {
            w(0) = i and mono_chain(xs, w, cmp, Nat.1)
        }
    }
}

/// The zero-th index of a cons list is its head.
theorem get_idx_cons_zero[T](head: T, tail: List[T]) {
    List.cons(head, tail).get_idx(0) = Option.some(head)
} by {
    if 0 > 0 {
        false
    }
    not 0 > 0
    List.cons(head, tail).get_idx(0) = Option.some(head)
}

/// The successor index of a cons list is the index of its tail.
theorem get_idx_cons_suc[T](head: T, tail: List[T], i: Nat) {
    List.cons(head, tail).get_idx(i.suc) = tail.get_idx(i)
} by {
    lt_add_suc(Nat.0, i)
    Nat.0 < Nat.0 + i.suc
    add_zero_left(i.suc)
    Nat.0 + i.suc = i.suc
    Nat.0 < i.suc
    i.suc > 0
    List.cons(head, tail).get_idx(i.suc) = tail.get_idx(i.suc - 1)
    suc_sub_one(i)
    i.suc - 1 = i
    List.cons(head, tail).get_idx(i.suc) = tail.get_idx(i)
}

/// The some-constructor is injective, as a boolean equivalence.
theorem option_some_eq_iff[T](x: T, y: T) {
    (Option.some(x) = Option.some(y)) = (x = y)
} by {
    if Option.some(x) = Option.some(y) {
        some_injective(x, y)
        x = y
    }
    if x = y {
        Option.some(x) = Option.some(y)
    }
    (Option.some(x) = Option.some(y)) = (x = y)
}

/// The value at position zero of a cons list is the head.
theorem value_at_cons_zero[T](head: T, tail: List[T], v: T) {
    value_at(List.cons(head, tail), 0, v) = (head = v)
} by {
    get_idx_cons_zero(head, tail)
    List.cons(head, tail).get_idx(0) = Option.some(head)
    value_at(List.cons(head, tail), 0, v) = (Option.some(head) = Option.some(v))
    option_some_eq_iff(head, v)
    (Option.some(head) = Option.some(v)) = (head = v)
    value_at(List.cons(head, tail), 0, v) = (head = v)
}

/// The value at a successor position of a cons list is the value in the tail.
theorem value_at_cons_suc[T](head: T, tail: List[T], i: Nat, v: T) {
    value_at(List.cons(head, tail), i.suc, v) = value_at(tail, i, v)
} by {
    get_idx_cons_suc(head, tail, i)
    List.cons(head, tail).get_idx(i.suc) = tail.get_idx(i)
    value_at(List.cons(head, tail), i.suc, v) = value_at(tail, i, v)
}

/// The value at a position is determined uniquely.
theorem value_at_det[T](xs: List[T], i: Nat, x: T, y: T) {
    value_at(xs, i, x) and value_at(xs, i, y) implies x = y
} by {
    if value_at(xs, i, x) and value_at(xs, i, y) {
        xs.get_idx(i) = Option.some(x)
        xs.get_idx(i) = Option.some(y)
        Option.some(x) = Option.some(y)
        some_injective(x, y)
        x = y
    }
}

/// Comparable values witness the comparison of positions.
theorem lt_at_of_values[T: LinearOrder](xs: List[T], i: Nat, j: Nat, x: T, y: T) {
    value_at(xs, i, x) and value_at(xs, j, y) and x < y implies lt_at(xs, i, j)
}

/// The comparison of positions is transitive.
theorem lt_at_trans[T: LinearOrder](xs: List[T], i: Nat, j: Nat, m: Nat) {
    lt_at(xs, i, j) and lt_at(xs, j, m) implies lt_at(xs, i, m)
} by {
    if lt_at(xs, i, j) and lt_at(xs, j, m) {
        let (x: T, y: T) satisfy {
            value_at(xs, i, x) and value_at(xs, j, y) and x < y
        }
        let (y2: T, z: T) satisfy {
            value_at(xs, j, y2) and value_at(xs, m, z) and y2 < z
        }
        value_at_det(xs, j, y, y2)
        y = y2
        y < z
        x < y and y < z
        lt_trans(x, y, z)
        x < z
        lt_at_of_values(xs, i, m, x, z)
        lt_at(xs, i, m)
    }
}

/// The transitivity of `lt_at` as a pointwise condition on a comparison function.
theorem lt_at_trans_forall[T: LinearOrder](xs: List[T]) {
    forall(u: Nat, v: Nat, m: Nat) {
        lt_at(xs, u, v) and lt_at(xs, v, m) implies lt_at(xs, u, m)
    }
}

/// The transitivity of `gt_at` as a pointwise condition on a comparison function.
theorem gt_at_trans_forall[T: LinearOrder](xs: List[T]) {
    forall(u: Nat, v: Nat, m: Nat) {
        gt_at(xs, u, v) and gt_at(xs, v, m) implies gt_at(xs, u, m)
    }
}

/// Every in-bounds position carries a value.
theorem get_idx_some_of_lt[T](xs: List[T], i: Nat) {
    i < xs.length implies exists(x: T) {
        xs.get_idx(i) = Option.some(x)
    }
} by {
    define p(l: List[T]) -> Bool {
        forall(idx: Nat) {
            idx < l.length implies exists(x: T) {
                l.get_idx(idx) = Option.some(x)
            }
        }
    }
    p(List.nil[T])
    forall(head: T, tail: List[T]) {
        if p(tail) {
            forall(idx: Nat) {
                if idx < List.cons(head, tail).length {
                    zero_or_suc(idx)
                    if idx = Nat.0 {
                        get_idx_cons_zero(head, tail)
                        exists(v: T) {
                            List.cons(head, tail).get_idx(idx) = Option.some(v)
                        }
                    }
                    if idx != Nat.0 {
                        let (j: Nat) satisfy { idx = j.suc }
                        j < tail.length
                        j < tail.length implies exists(v: T) {
                            tail.get_idx(j) = Option.some(v)
                        }
                        exists(v: T) {
                            tail.get_idx(j) = Option.some(v)
                        }
                        let (v: T) satisfy {
                            tail.get_idx(j) = Option.some(v)
                        }
                        get_idx_cons_suc(head, tail, j)
                        List.cons(head, tail).get_idx(idx) = Option.some(v)
                        exists(w2: T) {
                            List.cons(head, tail).get_idx(idx) = Option.some(w2)
                        }
                    }
                }
            }
            p(List.cons(head, tail))
        }
    }
    p(xs)
}

/// A chain starting at an in-bounds position is bounded in length by the list length.
theorem chain_length_bound[T: LinearOrder](xs: List[T], w: Nat -> Nat, cmp: Nat -> Nat -> Bool, k: Nat, i: Nat) {
    w(0) = i and i < xs.length and mono_chain(xs, w, cmp, k) implies k <= xs.length
} by {
    if w(0) = i and i < xs.length and mono_chain(xs, w, cmp, k) {
        forall(x: Nat, y: Nat) {
            if k.range.contains(x) and k.range.contains(y) and w(x) = w(y) {
                lt_of_range_contains(k, x)
                lt_of_range_contains(k, y)
                x < k
                y < k
                trichotomy(x, y)
                if x < y {
                    mono_chain_apply(xs, w, cmp, k, x, y)
                    x < y and y < k
                    w(x) < w(y)
                    lt_imp_ne(w(x), w(y))
                    w(x) != w(y)
                    false
                }
                if y < x {
                    mono_chain_apply(xs, w, cmp, k, y, x)
                    y < x and x < k
                    w(y) < w(x)
                    lt_imp_ne(w(y), w(x))
                    w(y) != w(x)
                    false
                }
                x = y
            }
        }
        locally_injective_map_is_unique(k.range, w)
        map(k.range, w).is_unique
        map(k.range, w).unique.length = map(k.range, w).length
        map(k.range, w).length = k.range.length
        k.range.length = k
        map(k.range, w).unique.length = k
        forall(y: Nat) {
            if map(k.range, w).contains(y) {
                map_contains(k.range, w, y)
                let (a: Nat) satisfy {
                    k.range.contains(a) and w(a) = y
                }
                lt_of_range_contains(k, a)
                a < k
                if a = Nat.0 {
                    w(a) = i
                    w(a) = y
                    y = i
                    i < xs.length
                    y < xs.length
                }
                if a != Nat.0 {
                    let (a0: Nat) satisfy { a = a0.suc }
                    lt_add_suc(Nat.0, a0)
                    Nat.0 < Nat.0 + a0.suc
                    add_zero_left(a0.suc)
                    Nat.0 + a0.suc = a0.suc
                    Nat.0 < a0.suc
                    a = a0.suc
                    Nat.0 < a
                    mono_chain_apply(xs, w, cmp, k, Nat.0, a)
                    Nat.0 < a and a < k
                    w(a) < xs.length
                    w(a) = y
                    y < xs.length
                }
                range_contains_of_lt(xs.length, y)
                xs.length.range.contains(y)
            }
        }
        unique_is_smallest_containing_list(map(k.range, w), xs.length.range)
        map(k.range, w).unique.length <= xs.length.range.length
        xs.length.range.length = xs.length
        map(k.range, w).unique.length <= xs.length
        k <= xs.length
    }
}

/// Prepends the position `i` to a witness chain, shifting the old positions up by one.
define prepend_pos(i: Nat, w: Nat -> Nat) -> Nat -> Nat {
    function(a: Nat) {
        if a = 0 {
            i
        } else {
            w(a - 1)
        }
    }
}

/// The prepended position is the first entry of the shifted chain.
theorem prepend_pos_zero(i: Nat, w: Nat -> Nat) {
    prepend_pos(i, w)(0) = i
}

/// The shifted chain at a successor position is the original chain.
theorem prepend_pos_suc(i: Nat, w: Nat -> Nat, a: Nat) {
    prepend_pos(i, w)(a.suc) = w(a)
} by {
    a.suc != Nat.0
    prepend_pos(i, w)(a.suc) = w(a.suc - 1)
    suc_sub_one(a)
    a.suc - 1 = a
    prepend_pos(i, w)(a.suc) = w(a)
}

/// Prepending at the very start of a singleton remainder.
theorem prepend_head_case_one[T: LinearOrder](xs: List[T], w: Nat -> Nat, cmp: Nat -> Nat -> Bool, k: Nat, i: Nat, j: Nat, a: Nat, b: Nat) {
    a = Nat.0 and b = Nat.1 and i < j and j < xs.length and cmp(i, j) and w(0) = j
    implies
    prepend_pos(i, w)(a) < prepend_pos(i, w)(b) and
    prepend_pos(i, w)(b) < xs.length and
    cmp(prepend_pos(i, w)(a), prepend_pos(i, w)(b))
} by {
    if a = Nat.0 and b = Nat.1 and i < j and j < xs.length and cmp(i, j) and w(0) = j {
        prepend_pos_zero(i, w)
        prepend_pos(i, w)(0) = i
        a = Nat.0
        prepend_pos(i, w)(a) = prepend_pos(i, w)(0)
        prepend_pos(i, w)(a) = i
        prepend_pos_suc(i, w, Nat.0)
        prepend_pos(i, w)(Nat.1) = w(Nat.0)
        b = Nat.1
        prepend_pos(i, w)(b) = prepend_pos(i, w)(Nat.1)
        prepend_pos(i, w)(b) = w(Nat.0)
        w(0) = j
        prepend_pos(i, w)(b) = j
        i < j
        prepend_pos(i, w)(a) < prepend_pos(i, w)(b)
        j < xs.length
        prepend_pos(i, w)(b) < xs.length
        cmp(i, j)
        cmp(prepend_pos(i, w)(a), prepend_pos(i, w)(b))
        prepend_pos(i, w)(a) < prepend_pos(i, w)(b) and
            prepend_pos(i, w)(b) < xs.length and
            cmp(prepend_pos(i, w)(a), prepend_pos(i, w)(b))
    }
}

/// Prepending at the start with a remainder of length at least two.
theorem prepend_head_case_ge_two[T: LinearOrder](xs: List[T], w: Nat -> Nat, cmp: Nat -> Nat -> Bool, k: Nat, i: Nat, j: Nat, a: Nat, b: Nat, b0: Nat) {
    a = Nat.0 and b = b0.suc and b0 != Nat.0 and b < k.suc and i < j and j < xs.length and
    cmp(i, j) and w(0) = j and mono_chain(xs, w, cmp, k) and
    (forall(u: Nat, v: Nat, m: Nat) { cmp(u, v) and cmp(v, m) implies cmp(u, m) })
    implies
    prepend_pos(i, w)(a) < prepend_pos(i, w)(b) and
    prepend_pos(i, w)(b) < xs.length and
    cmp(prepend_pos(i, w)(a), prepend_pos(i, w)(b))
} by {
    if a = Nat.0 and b = b0.suc and b0 != Nat.0 and b < k.suc and i < j and j < xs.length and
        cmp(i, j) and w(0) = j and mono_chain(xs, w, cmp, k) and
        (forall(u: Nat, v: Nat, m: Nat) { cmp(u, v) and cmp(v, m) implies cmp(u, m) }) {
        prepend_pos_zero(i, w)
        prepend_pos(i, w)(0) = i
        a = Nat.0
        prepend_pos(i, w)(a) = prepend_pos(i, w)(0)
        prepend_pos(i, w)(a) = i
        prepend_pos_suc(i, w, b0)
        prepend_pos(i, w)(b0.suc) = w(b0)
        b = b0.suc
        prepend_pos(i, w)(b) = w(b0)
        zero_or_suc(b0)
        let (c: Nat) satisfy { b0 = c.suc }
        lt_add_suc(Nat.0, c)
        Nat.0 < Nat.0 + c.suc
        add_zero_left(c.suc)
        Nat.0 + c.suc = c.suc
        Nat.0 < c.suc
        b0 = c.suc
        Nat.0 < b0
        lt_cancel_suc(b0, k)
        b0 < k
        mono_chain_apply(xs, w, cmp, k, Nat.0, b0)
        Nat.0 < b0 and b0 < k
        w(0) < w(b0)
        w(b0) < xs.length
        cmp(w(0), w(b0))
        w(0) = j
        j < w(b0)
        i < j
        lt_trans(i, j, w(b0))
        i < w(b0)
        prepend_pos(i, w)(a) < prepend_pos(i, w)(b)
        prepend_pos(i, w)(b) < xs.length
        cmp(j, w(b0))
        cmp(i, j) and cmp(j, w(b0))
        (cmp(i, j) and cmp(j, w(b0)) implies cmp(i, w(b0)))
        cmp(i, w(b0))
        prepend_pos(i, w)(a) = i
        cmp(prepend_pos(i, w)(a), prepend_pos(i, w)(b))
        prepend_pos(i, w)(a) < prepend_pos(i, w)(b) and
            prepend_pos(i, w)(b) < xs.length and
            cmp(prepend_pos(i, w)(a), prepend_pos(i, w)(b))
    }
}

/// Prepending when the first position is not the head.
theorem prepend_suc_case[T: LinearOrder](xs: List[T], w: Nat -> Nat, cmp: Nat -> Nat -> Bool, k: Nat, i: Nat, j: Nat, a: Nat, b: Nat, a0: Nat, b0: Nat) {
    a = a0.suc and b = b0.suc and a0 < b0 and b0 < k and mono_chain(xs, w, cmp, k)
    implies
    prepend_pos(i, w)(a) < prepend_pos(i, w)(b) and
    prepend_pos(i, w)(b) < xs.length and
    cmp(prepend_pos(i, w)(a), prepend_pos(i, w)(b))
} by {
    if a = a0.suc and b = b0.suc and a0 < b0 and b0 < k and mono_chain(xs, w, cmp, k) {
        prepend_pos_suc(i, w, a0)
        prepend_pos(i, w)(a) = w(a0)
        prepend_pos_suc(i, w, b0)
        prepend_pos(i, w)(b) = w(b0)
        mono_chain_apply(xs, w, cmp, k, a0, b0)
        a0 < b0 and b0 < k
        w(a0) < w(b0)
        w(b0) < xs.length
        cmp(w(a0), w(b0))
        prepend_pos(i, w)(a) < prepend_pos(i, w)(b)
        prepend_pos(i, w)(b) < xs.length
        cmp(prepend_pos(i, w)(a), prepend_pos(i, w)(b))
        prepend_pos(i, w)(a) < prepend_pos(i, w)(b) and
            prepend_pos(i, w)(b) < xs.length and
            cmp(prepend_pos(i, w)(a), prepend_pos(i, w)(b))
    }
}

/// Prepending at the start of the chain, any remainder length.
theorem prepend_head_any_case[T: LinearOrder](xs: List[T], w: Nat -> Nat, cmp: Nat -> Nat -> Bool, k: Nat, i: Nat, j: Nat, a: Nat, b: Nat) {
    a = Nat.0 and i < j and j < xs.length and cmp(i, j) and w(0) = j and
    mono_chain(xs, w, cmp, k) and a < b and b < k.suc and
    (forall(u: Nat, v: Nat, m: Nat) { cmp(u, v) and cmp(v, m) implies cmp(u, m) })
    implies
    prepend_pos(i, w)(a) < prepend_pos(i, w)(b) and
    prepend_pos(i, w)(b) < xs.length and
    cmp(prepend_pos(i, w)(a), prepend_pos(i, w)(b))
} by {
    if a = Nat.0 and i < j and j < xs.length and cmp(i, j) and w(0) = j and
        mono_chain(xs, w, cmp, k) and a < b and b < k.suc and
        (forall(u: Nat, v: Nat, m: Nat) { cmp(u, v) and cmp(v, m) implies cmp(u, m) }) {
        lt_imp_ne(a, b)
        a != b
        b != Nat.0
        zero_or_suc(b)
        let (b0: Nat) satisfy { b = b0.suc }
        if b0 = Nat.0 {
            prepend_head_case_one(xs, w, cmp, k, i, j, a, b)
            prepend_pos(i, w)(a) < prepend_pos(i, w)(b) and
                prepend_pos(i, w)(b) < xs.length and
                cmp(prepend_pos(i, w)(a), prepend_pos(i, w)(b))
        }
        if b0 != Nat.0 {
            prepend_head_case_ge_two(xs, w, cmp, k, i, j, a, b, b0)
            prepend_pos(i, w)(a) < prepend_pos(i, w)(b) and
                prepend_pos(i, w)(b) < xs.length and
                cmp(prepend_pos(i, w)(a), prepend_pos(i, w)(b))
        }
        prepend_pos(i, w)(a) < prepend_pos(i, w)(b) and
            prepend_pos(i, w)(b) < xs.length and
            cmp(prepend_pos(i, w)(a), prepend_pos(i, w)(b))
    }
}

/// Prepending away from the head of the chain.
theorem prepend_suc_any_case[T: LinearOrder](xs: List[T], w: Nat -> Nat, cmp: Nat -> Nat -> Bool, k: Nat, i: Nat, j: Nat, a: Nat, b: Nat) {
    a != Nat.0 and i < j and j < xs.length and cmp(i, j) and w(0) = j and
    mono_chain(xs, w, cmp, k) and a < b and b < k.suc and
    (forall(u: Nat, v: Nat, m: Nat) { cmp(u, v) and cmp(v, m) implies cmp(u, m) })
    implies
    prepend_pos(i, w)(a) < prepend_pos(i, w)(b) and
    prepend_pos(i, w)(b) < xs.length and
    cmp(prepend_pos(i, w)(a), prepend_pos(i, w)(b))
} by {
    if a != Nat.0 and i < j and j < xs.length and cmp(i, j) and w(0) = j and
        mono_chain(xs, w, cmp, k) and a < b and b < k.suc and
        (forall(u: Nat, v: Nat, m: Nat) { cmp(u, v) and cmp(v, m) implies cmp(u, m) }) {
        zero_or_suc(a)
        let (a0: Nat) satisfy { a = a0.suc }
        lt_imp_ne(a, b)
        a != b
        b != Nat.0
        zero_or_suc(b)
        let (b0: Nat) satisfy { b = b0.suc }
        lt_cancel_suc(a0, b0)
        a0 < b0
        lt_cancel_suc(b0, k)
        b0 < k
        prepend_suc_case(xs, w, cmp, k, i, j, a, b, a0, b0)
        prepend_pos(i, w)(a) < prepend_pos(i, w)(b) and
            prepend_pos(i, w)(b) < xs.length and
            cmp(prepend_pos(i, w)(a), prepend_pos(i, w)(b))
    }
}

/// Prepending a `cmp`-comparable position to a chain yields a chain.
theorem mono_chain_prepend[T: LinearOrder](xs: List[T], w: Nat -> Nat, cmp: Nat -> Nat -> Bool, k: Nat, i: Nat, j: Nat) {
    i < j and j < xs.length and cmp(i, j) and w(0) = j and mono_chain(xs, w, cmp, k) and
    (forall(u: Nat, v: Nat, m: Nat) { cmp(u, v) and cmp(v, m) implies cmp(u, m) })
    implies mono_chain(xs, prepend_pos(i, w), cmp, k.suc)
} by {
    if i < j and j < xs.length and cmp(i, j) and w(0) = j and mono_chain(xs, w, cmp, k) and
        (forall(u: Nat, v: Nat, m: Nat) { cmp(u, v) and cmp(v, m) implies cmp(u, m) }) {
        forall(a: Nat, b: Nat) {
            if a < b and b < k.suc {
                zero_or_suc(a)
                if a = Nat.0 {
                    prepend_head_any_case(xs, w, cmp, k, i, j, a, b)
                    prepend_pos(i, w)(a) < prepend_pos(i, w)(b) and
                        prepend_pos(i, w)(b) < xs.length and
                        cmp(prepend_pos(i, w)(a), prepend_pos(i, w)(b))
                }
                if a != Nat.0 {
                    prepend_suc_any_case(xs, w, cmp, k, i, j, a, b)
                    prepend_pos(i, w)(a) < prepend_pos(i, w)(b) and
                        prepend_pos(i, w)(b) < xs.length and
                        cmp(prepend_pos(i, w)(a), prepend_pos(i, w)(b))
                }
                prepend_pos(i, w)(a) < prepend_pos(i, w)(b) and
                    prepend_pos(i, w)(b) < xs.length and
                    cmp(prepend_pos(i, w)(a), prepend_pos(i, w)(b))
            }
        }
    }
}

/// A chain starting at `j` extends to a chain starting at an earlier comparable `i`.
theorem chain_geq_from_prepend[T: LinearOrder](xs: List[T], cmp: Nat -> Nat -> Bool, i: Nat, j: Nat, k: Nat) {
    i < j and j < xs.length and cmp(i, j) and chain_geq_from(xs, cmp, j, k) and
    (forall(u: Nat, v: Nat, m: Nat) { cmp(u, v) and cmp(v, m) implies cmp(u, m) })
    implies chain_geq_from(xs, cmp, i, k.suc)
} by {
    if i < j and j < xs.length and cmp(i, j) and chain_geq_from(xs, cmp, j, k) and
        (forall(u: Nat, v: Nat, m: Nat) { cmp(u, v) and cmp(v, m) implies cmp(u, m) }) {
        exists(w: Nat -> Nat) {
            w(0) = j and mono_chain(xs, w, cmp, k)
        }
        let (w: Nat -> Nat) satisfy {
            w(0) = j and mono_chain(xs, w, cmp, k)
        }
        mono_chain_prepend(xs, w, cmp, k, i, j)
        mono_chain(xs, prepend_pos(i, w), cmp, k.suc)
        prepend_pos_zero(i, w)
        prepend_pos(i, w)(0) = i
        exists(v: Nat -> Nat) {
            v(0) = i and mono_chain(xs, v, cmp, k.suc)
        }
    }
}

/// The chain lengths starting at an in-bounds position are bounded by the list length.
theorem chain_geq_upper_bound[T: LinearOrder](xs: List[T], cmp: Nat -> Nat -> Bool, i: Nat) {
    i < xs.length implies is_upper_bound_of(chain_geq_from(xs, cmp, i), xs.length)
} by {
    if i < xs.length {
        forall(k: Nat) {
            if chain_geq_from(xs, cmp, i, k) {
                exists(w: Nat -> Nat) {
                    w(0) = i and mono_chain(xs, w, cmp, k)
                }
                let (w: Nat -> Nat) satisfy {
                    w(0) = i and mono_chain(xs, w, cmp, k)
                }
                chain_length_bound(xs, w, cmp, k, i)
                w(0) = i and i < xs.length and mono_chain(xs, w, cmp, k)
                k <= xs.length
            }
            chain_geq_from(xs, cmp, i, k) implies k <= xs.length
        }
        is_upper_bound_of_intro(chain_geq_from(xs, cmp, i), xs.length)
        is_upper_bound_of(chain_geq_from(xs, cmp, i), xs.length)
    }
}

/// The length of the longest `cmp`-chain starting at position `i` (0 if out of bounds).
let chain_val[T: LinearOrder](xs: List[T], cmp: Nat -> Nat -> Bool, i: Nat) -> m: Nat satisfy {
    (i < xs.length and is_max(chain_geq_from(xs, cmp, i), m)) or
    (not i < xs.length and m = Nat.0)
} by {
    if i < xs.length {
        chain_geq_from(xs, cmp, i, Nat.0)
        chain_geq_upper_bound(xs, cmp, i)
        is_upper_bound_of(chain_geq_from(xs, cmp, i), xs.length)
        has_max(chain_geq_from(xs, cmp, i), Nat.0, xs.length)
        exists(m: Nat) {
            is_max(chain_geq_from(xs, cmp, i), m)
        }
        let (m: Nat) satisfy {
            is_max(chain_geq_from(xs, cmp, i), m)
        }
        is_max(chain_geq_from(xs, cmp, i), m)
        i < xs.length and is_max(chain_geq_from(xs, cmp, i), m)
        (i < xs.length and is_max(chain_geq_from(xs, cmp, i), m)) or
            (not i < xs.length and m = Nat.0)
    }
    if not i < xs.length {
        not i < xs.length and Nat.0 = Nat.0
        (i < xs.length and is_max(chain_geq_from(xs, cmp, i), Nat.0)) or
            (not i < xs.length and Nat.0 = Nat.0)
    }
}

/// The chain length at an in-bounds position is the maximum chain length.
theorem chain_val_apply[T: LinearOrder](xs: List[T], cmp: Nat -> Nat -> Bool, i: Nat) {
    i < xs.length implies is_max(chain_geq_from(xs, cmp, i), chain_val(xs, cmp, i))
} by {
    if i < xs.length {
        (i < xs.length and is_max(chain_geq_from(xs, cmp, i), chain_val(xs, cmp, i))) or
            (not i < xs.length and chain_val(xs, cmp, i) = Nat.0)
        i < xs.length and is_max(chain_geq_from(xs, cmp, i), chain_val(xs, cmp, i))
        is_max(chain_geq_from(xs, cmp, i), chain_val(xs, cmp, i))
    }
}

/// Every chain length is at least one at an in-bounds position.
theorem chain_val_geq_one[T: LinearOrder](xs: List[T], cmp: Nat -> Nat -> Bool, i: Nat) {
    i < xs.length implies Nat.1 <= chain_val(xs, cmp, i)
} by {
    if i < xs.length {
        chain_geq_from_one(xs, cmp, i)
        chain_geq_from(xs, cmp, i, Nat.1)
        chain_val_apply(xs, cmp, i)
        is_max(chain_geq_from(xs, cmp, i), chain_val(xs, cmp, i))
        is_max_is_upper_bound(chain_geq_from(xs, cmp, i), chain_val(xs, cmp, i), Nat.1)
        Nat.1 <= chain_val(xs, cmp, i)
    }
}

/// An earlier comparable position extends every chain starting at the later one.
theorem chain_val_step[T: LinearOrder](xs: List[T], cmp: Nat -> Nat -> Bool, i: Nat, j: Nat) {
    i < j and j < xs.length and cmp(i, j) and
    (forall(u: Nat, v: Nat, m: Nat) { cmp(u, v) and cmp(v, m) implies cmp(u, m) })
    implies chain_val(xs, cmp, j).suc <= chain_val(xs, cmp, i)
} by {
    if i < j and j < xs.length and cmp(i, j) and
        (forall(u: Nat, v: Nat, m: Nat) { cmp(u, v) and cmp(v, m) implies cmp(u, m) }) {
        chain_val_apply(xs, cmp, j)
        is_max(chain_geq_from(xs, cmp, j), chain_val(xs, cmp, j))
        is_max_apply(chain_geq_from(xs, cmp, j), chain_val(xs, cmp, j))
        chain_geq_from(xs, cmp, j, chain_val(xs, cmp, j))
        chain_geq_from_prepend(xs, cmp, i, j, chain_val(xs, cmp, j))
        chain_geq_from(xs, cmp, i, chain_val(xs, cmp, j).suc)
        chain_val_apply(xs, cmp, i)
        is_max(chain_geq_from(xs, cmp, i), chain_val(xs, cmp, i))
        is_max_is_upper_bound(chain_geq_from(xs, cmp, i), chain_val(xs, cmp, i), chain_val(xs, cmp, j).suc)
        chain_val(xs, cmp, j).suc <= chain_val(xs, cmp, i)
    }
}

/// Without a chain of length `k + 1`, every chain length is at most `k`.
theorem chain_val_bound[T: LinearOrder](xs: List[T], cmp: Nat -> Nat -> Bool, i: Nat, k: Nat) {
    i < xs.length and not chain_geq(xs, cmp, k.suc) implies chain_val(xs, cmp, i) <= k
} by {
    if i < xs.length and not chain_geq(xs, cmp, k.suc) {
        lt_or_lte(k, chain_val(xs, cmp, i))
        if k < chain_val(xs, cmp, i) {
            lt_imp_lte_suc(k, chain_val(xs, cmp, i))
            k.suc <= chain_val(xs, cmp, i)
            chain_val_apply(xs, cmp, i)
            is_max(chain_geq_from(xs, cmp, i), chain_val(xs, cmp, i))
            is_max_apply(chain_geq_from(xs, cmp, i), chain_val(xs, cmp, i))
            chain_geq_from(xs, cmp, i, chain_val(xs, cmp, i))
            exists(w: Nat -> Nat) {
                w(0) = i and mono_chain(xs, w, cmp, chain_val(xs, cmp, i))
            }
            let (w: Nat -> Nat) satisfy {
                w(0) = i and mono_chain(xs, w, cmp, chain_val(xs, cmp, i))
            }
            mono_chain_mono(xs, w, cmp, k.suc, chain_val(xs, cmp, i))
            k.suc <= chain_val(xs, cmp, i)
            mono_chain(xs, w, cmp, k.suc)
            exists(w2: Nat -> Nat) {
                mono_chain(xs, w2, cmp, k.suc)
            }
            chain_geq(xs, cmp, k.suc)
            false
        }
        if chain_val(xs, cmp, i) <= k {
            chain_val(xs, cmp, i) <= k
        }
        chain_val(xs, cmp, i) <= k
    }
}

/// A position that carries a value witnesses containment.
theorem value_at_imp_contains[T](xs: List[T], i: Nat, x: T) {
    value_at(xs, i, x) implies xs.contains(x)
} by {
    define p(l: List[T]) -> Bool {
        forall(idx: Nat, v: T) {
            value_at(l, idx, v) implies l.contains(v)
        }
    }
    forall(idx: Nat, v: T) {
        if value_at(List.nil[T], idx, v) {
            List.nil[T].get_idx(idx) = Option.none
            Option.none = Option.some(v)
            false
        }
    }
    p(List.nil[T])
    forall(head: T, tail: List[T]) {
        if p(tail) {
            forall(idx: Nat, v: T) {
                if value_at(List.cons(head, tail), idx, v) {
                    if idx = Nat.0 {
                        value_at_cons_zero(head, tail, v)
                        head = v
                        List.cons(head, tail).contains(v)
                    }
                    if idx != Nat.0 {
                        let (j: Nat) satisfy { idx = j.suc }
                        value_at_cons_suc(head, tail, j, v)
                        value_at(tail, j, v)
                        value_at(tail, j, v) implies tail.contains(v)
                        tail.contains(v)
                        List.cons(head, tail).contains(v)
                    }
                }
            }
            p(List.cons(head, tail))
        }
    }
    p(xs)
}

/// A position that carries a value counts at least one.
theorem count_ge_one_of_value_at[T](xs: List[T], i: Nat, x: T) {
    value_at(xs, i, x) implies xs.count(x) >= Nat.1
} by {
    if value_at(xs, i, x) {
        value_at_imp_contains(xs, i, x)
        xs.contains(x)
        list_contains_implies_count_geq_one(xs, x)
        xs.count(x) >= Nat.1
    }
}

/// Two distinct positions carrying the same value force the count to be at least two.
theorem count_ge_two_of_two_positions[T](xs: List[T], i: Nat, j: Nat, x: T) {
    value_at(xs, i, x) and value_at(xs, j, x) and i != j implies xs.count(x) >= Nat.2
} by {
    define p(l: List[T]) -> Bool {
        forall(a: Nat, b: Nat) {
            value_at(l, a, x) and value_at(l, b, x) and a != b implies l.count(x) >= Nat.2
        }
    }
    forall(a: Nat, b: Nat) {
        if value_at(List.nil[T], a, x) and value_at(List.nil[T], b, x) and a != b {
            List.nil[T].get_idx(a) = Option.none
            Option.none = Option.some(x)
            false
        }
    }
    p(List.nil[T])
    forall(head: T, tail: List[T]) {
        if p(tail) {
            forall(a: Nat, b: Nat) {
                if value_at(List.cons(head, tail), a, x) and value_at(List.cons(head, tail), b, x) and a != b {
                    if head = x {
                        zero_or_suc(a)
                        if a = Nat.0 {
                            a != b
                            b != Nat.0
                            zero_or_suc(b)
                            let (b0: Nat) satisfy { b = b0.suc }
                            value_at_cons_suc(head, tail, b0, x)
                            value_at(tail, b0, x)
                            count_ge_one_of_value_at(tail, b0, x)
                            tail.count(x) >= Nat.1
                        }
                        if a != Nat.0 {
                            let (a0: Nat) satisfy { a = a0.suc }
                            value_at_cons_suc(head, tail, a0, x)
                            value_at(tail, a0, x)
                            count_ge_one_of_value_at(tail, a0, x)
                            tail.count(x) >= Nat.1
                        }
                        tail.count(x) >= Nat.1
                        List.cons(head, tail).count(x) = 1 + tail.count(x)
                        lte_add_left(Nat.1, Nat.1, tail.count(x))
                        Nat.1 + Nat.1 <= Nat.1 + tail.count(x)
                        List.cons(head, tail).count(x) >= Nat.2
                    }
                    if head != x {
                        if a = Nat.0 {
                            value_at_cons_zero(head, tail, x)
                            head = x
                            head != x
                            false
                        }
                        a != Nat.0
                        zero_or_suc(a)
                        let (a0: Nat) satisfy { a = a0.suc }
                        value_at_cons_suc(head, tail, a0, x)
                        value_at(tail, a0, x)
                        if b = Nat.0 {
                            value_at_cons_zero(head, tail, x)
                            head = x
                            head != x
                            false
                        }
                        b != Nat.0
                        zero_or_suc(b)
                        let (b0: Nat) satisfy { b = b0.suc }
                        if a0 = b0 {
                            a = a0.suc
                            b = b0.suc
                            a = b
                            a != b
                            false
                        }
                        a0 != b0
                        value_at_cons_suc(head, tail, b0, x)
                        value_at(tail, b0, x)
                        (value_at(tail, a0, x) and value_at(tail, b0, x) and a0 != b0 implies tail.count(x) >= Nat.2)
                        value_at(tail, a0, x) and value_at(tail, b0, x) and a0 != b0
                        tail.count(x) >= Nat.2
                        List.cons(head, tail).count(x) = tail.count(x)
                        List.cons(head, tail).count(x) >= Nat.2
                    }
                    List.cons(head, tail).count(x) >= Nat.2
                }
            }
            p(List.cons(head, tail))
        }
    }
    p(xs)
}

/// A unique list carries distinct values at distinct positions.
theorem is_unique_distinct_values[T](xs: List[T], i: Nat, j: Nat) {
    xs.is_unique and i < xs.length and j < xs.length and i != j implies
    not exists(x: T) { value_at(xs, i, x) and value_at(xs, j, x) }
} by {
    if xs.is_unique and i < xs.length and j < xs.length and i != j {
        if exists(x: T) { value_at(xs, i, x) and value_at(xs, j, x) } {
            let (x: T) satisfy { value_at(xs, i, x) and value_at(xs, j, x) }
            count_ge_two_of_two_positions(xs, i, j, x)
            value_at(xs, i, x) and value_at(xs, j, x) and i != j
            xs.count(x) >= Nat.2
            unique_implies_no_duplicate(xs, x)
            xs.count(x) <= Nat.1
            Nat.2 <= xs.count(x)
            lt_suc(Nat.1)
            Nat.1 < Nat.2
            lte_and_lt(xs.count(x), Nat.1, Nat.2)
            xs.count(x) < Nat.2
            lte_and_lt(Nat.2, xs.count(x), Nat.2)
            Nat.2 < Nat.2
            lt_not_ref(Nat.2)
            false
        }
        not exists(x: T) { value_at(xs, i, x) and value_at(xs, j, x) }
    }
}

/// Distinct in-bounds positions are strictly comparable one way or the other.
theorem lt_at_total[T: LinearOrder](xs: List[T], i: Nat, j: Nat) {
    xs.is_unique and i < j and j < xs.length implies lt_at(xs, i, j) or lt_at(xs, j, i)
} by {
    if xs.is_unique and i < j and j < xs.length {
        i < xs.length
        get_idx_some_of_lt(xs, i)
        exists(x: T) { xs.get_idx(i) = Option.some(x) }
        let (x: T) satisfy { xs.get_idx(i) = Option.some(x) }
        get_idx_some_of_lt(xs, j)
        exists(y: T) { xs.get_idx(j) = Option.some(y) }
        let (y: T) satisfy { xs.get_idx(j) = Option.some(y) }
        value_at(xs, i, x)
        value_at(xs, j, y)
        if x = y {
            is_unique_distinct_values(xs, i, j)
            exists(z: T) { value_at(xs, i, z) and value_at(xs, j, z) }
            false
        }
        x != y
        lte_or_lte_swap(x, y)
        x <= y or y <= x
        if x <= y {
            if x = y {
                false
            }
            x != y
            x < y
            lt_at_of_values(xs, i, j, x, y)
            lt_at(xs, i, j)
            lt_at(xs, i, j) or lt_at(xs, j, i)
        }
        if y <= x {
            if y = x {
                false
            }
            y != x
            y < x
            lt_at_of_values(xs, j, i, y, x)
            lt_at(xs, j, i)
            lt_at(xs, i, j) or lt_at(xs, j, i)
        }
        lt_at(xs, i, j) or lt_at(xs, j, i)
    }
}

/// The Erdős–Szekeres pair assigned to a position: (increasing length, decreasing length).
define esz_pair[T: LinearOrder](xs: List[T], i: Nat) -> Pair[Nat, Nat] {
    Pair.new(chain_val(xs, lt_at(xs), i), chain_val(xs, gt_at(xs), i))
}

/// The map from positions to their Erdős–Szekeres pairs.
define esz_pair_fn[T: LinearOrder](xs: List[T], idx: Nat) -> Pair[Nat, Nat] {
    esz_pair(xs, idx)
}

/// The first projection of an Erdős–Szekeres pair is the increasing chain length.
theorem esz_pair_first[T: LinearOrder](xs: List[T], i: Nat) {
    esz_pair(xs, i).first = chain_val(xs, lt_at(xs), i)
} by {
    esz_pair(xs, i) = Pair.new(chain_val(xs, lt_at(xs), i), chain_val(xs, gt_at(xs), i))
    pair_new_first(chain_val(xs, lt_at(xs), i), chain_val(xs, gt_at(xs), i))
    Pair.new(chain_val(xs, lt_at(xs), i), chain_val(xs, gt_at(xs), i)).first = chain_val(xs, lt_at(xs), i)
    esz_pair(xs, i).first = chain_val(xs, lt_at(xs), i)
}

/// The second projection of an Erdős–Szekeres pair is the decreasing chain length.
theorem esz_pair_second[T: LinearOrder](xs: List[T], i: Nat) {
    esz_pair(xs, i).second = chain_val(xs, gt_at(xs), i)
} by {
    esz_pair(xs, i) = Pair.new(chain_val(xs, lt_at(xs), i), chain_val(xs, gt_at(xs), i))
    pair_new_second(chain_val(xs, lt_at(xs), i), chain_val(xs, gt_at(xs), i))
    Pair.new(chain_val(xs, lt_at(xs), i), chain_val(xs, gt_at(xs), i)).second = chain_val(xs, gt_at(xs), i)
    esz_pair(xs, i).second = chain_val(xs, gt_at(xs), i)
}

/// Distinct positions carry distinct Erdős–Szekeres pairs.
theorem esz_pair_distinct[T: LinearOrder](xs: List[T], i: Nat, j: Nat) {
    xs.is_unique and i < j and j < xs.length implies esz_pair(xs, i) != esz_pair(xs, j)
} by {
    if xs.is_unique and i < j and j < xs.length {
        lt_at_total(xs, i, j)
        lt_at(xs, i, j) or lt_at(xs, j, i)
        if lt_at(xs, i, j) {
            lt_at_trans_forall(xs)
            chain_val_step(xs, lt_at(xs), i, j)
            chain_val(xs, lt_at(xs), j).suc <= chain_val(xs, lt_at(xs), i)
            lt_suc(chain_val(xs, lt_at(xs), j))
            chain_val(xs, lt_at(xs), j) < chain_val(xs, lt_at(xs), j).suc
            lt_and_lte(chain_val(xs, lt_at(xs), j), chain_val(xs, lt_at(xs), j).suc, chain_val(xs, lt_at(xs), i))
            chain_val(xs, lt_at(xs), j) < chain_val(xs, lt_at(xs), i)
            esz_pair(xs, i).first = chain_val(xs, lt_at(xs), i)
            esz_pair(xs, j).first = chain_val(xs, lt_at(xs), j)
            esz_pair(xs, j).first < esz_pair(xs, i).first
            if esz_pair(xs, i) = esz_pair(xs, j) {
                esz_pair(xs, i).first = esz_pair(xs, j).first
                esz_pair(xs, j).first < esz_pair(xs, j).first
                lt_not_ref(esz_pair(xs, j).first)
                false
            }
            esz_pair(xs, i) != esz_pair(xs, j)
        }
        if lt_at(xs, j, i) {
            gt_at_trans_forall(xs)
            gt_at(xs, i, j) = lt_at(xs, j, i)
            chain_val_step(xs, gt_at(xs), i, j)
            chain_val(xs, gt_at(xs), j).suc <= chain_val(xs, gt_at(xs), i)
            lt_suc(chain_val(xs, gt_at(xs), j))
            chain_val(xs, gt_at(xs), j) < chain_val(xs, gt_at(xs), j).suc
            lt_and_lte(chain_val(xs, gt_at(xs), j), chain_val(xs, gt_at(xs), j).suc, chain_val(xs, gt_at(xs), i))
            chain_val(xs, gt_at(xs), j) < chain_val(xs, gt_at(xs), i)
            esz_pair(xs, i).second = chain_val(xs, gt_at(xs), i)
            esz_pair(xs, j).second = chain_val(xs, gt_at(xs), j)
            esz_pair(xs, j).second < esz_pair(xs, i).second
            if esz_pair(xs, i) = esz_pair(xs, j) {
                esz_pair(xs, i).second = esz_pair(xs, j).second
                esz_pair(xs, j).second < esz_pair(xs, j).second
                lt_not_ref(esz_pair(xs, j).second)
                false
            }
            esz_pair(xs, i) != esz_pair(xs, j)
        }
        esz_pair(xs, i) != esz_pair(xs, j)
    }
}

/// The map shifting a natural up by one, used for the box rows.
define shift_one(a: Nat) -> Nat {
    a + 1
}

/// The pair in the `r`-th row of the box.
define box_pair(r: Nat, b: Nat) -> Pair[Nat, Nat] {
    Pair.new(r, shift_one(b))
}

/// The box `{1, ..., r} x {1, ..., s}` of pairs, as a list.
define pair_boxes(r: Nat, s: Nat) -> List[Pair[Nat, Nat]] {
    match r {
        Nat.zero {
            List.nil[Pair[Nat, Nat]]
        }
        Nat.suc(p) {
            pair_boxes(p, s) + map(s.range, box_pair(r))
        }
    }
}

/// Shifting down and up by one recovers the original number.
theorem shift_one_pred(a: Nat) {
    Nat.1 <= a implies shift_one(a - 1) = a
} by {
    if Nat.1 <= a {
        add_sub(a, Nat.1)
        a - 1 + 1 = a
        shift_one(a - 1) = a
    }
}

/// The predecessor of a positive number below `b` is below `b`.
theorem pred_lt_of_lte(a: Nat, b: Nat) {
    Nat.1 <= a and a <= b implies a - 1 < b
} by {
    if Nat.1 <= a and a <= b {
        add_sub(a, Nat.1)
        a - 1 + 1 = a
        lt_suc(a - 1)
        a - 1 < a - 1 + 1
        a - 1 < a
        lt_and_lte(a - 1, a, b)
        a - 1 < b
    }
}

/// The box has exactly `r * s` pairs.
theorem pair_boxes_length(r: Nat, s: Nat) {
    pair_boxes(r, s).length = r * s
} by {
    define p(n: Nat) -> Bool {
        pair_boxes(n, s).length = n * s
    }
    pair_boxes(Nat.0, s).length = Nat.0
    p(Nat.0)
    forall(n: Nat) {
        if p(n) {
            pair_boxes(n.suc, s) = pair_boxes(n, s) + map(s.range, box_pair(n.suc))
            add_length(pair_boxes(n, s), map(s.range, box_pair(n.suc)))
            pair_boxes(n, s).length + map(s.range, box_pair(n.suc)).length = (pair_boxes(n, s) + map(s.range, box_pair(n.suc))).length
            map_length(s.range, box_pair(n.suc))
            map(s.range, box_pair(n.suc)).length = s.range.length
            length_range(s)
            s.range.length = s
            map(s.range, box_pair(n.suc)).length = s
            pair_boxes(n, s).length = n * s
            pair_boxes(n.suc, s).length = n * s + s
            mul_suc_right(n, s)
            n * s.suc = n * s + n
            n.suc * s = n * s + s
            pair_boxes(n.suc, s).length = n.suc * s
            p(n.suc)
        }
    }
    p(r)
}

/// Every pair with a first component in `{1, ..., r}` and a second in `{1, ..., s}` lies in the box.
theorem pair_boxes_contains(r: Nat, s: Nat, a: Nat, b: Nat) {
    Nat.1 <= a and a <= r and Nat.1 <= b and b <= s implies
    pair_boxes(r, s).contains(Pair.new(a, b))
} by {
    define p(n: Nat) -> Bool {
        forall(c: Nat, d: Nat) {
            Nat.1 <= c and c <= n and Nat.1 <= d and d <= s implies
            pair_boxes(n, s).contains(Pair.new(c, d))
        }
    }
    forall(c: Nat, d: Nat) {
        if Nat.1 <= c and c <= Nat.0 and Nat.1 <= d and d <= s {
            only_zero_lte_zero(c)
            c = Nat.0
            Nat.1 <= c
            Nat.1 <= Nat.0
            false
        }
    }
    p(Nat.0)
    forall(n: Nat) {
        if p(n) {
            forall(c: Nat, d: Nat) {
                if Nat.1 <= c and c <= n.suc and Nat.1 <= d and d <= s {
                    if c = n.suc {
                        shift_one_pred(d)
                        Nat.1 <= d
                        shift_one(d - 1) = d
                        pred_lt_of_lte(d, s)
                        Nat.1 <= d and d <= s
                        d - 1 < s
                        range_contains_of_lt(s, d - 1)
                        s.range.contains(d - 1)
                        map_contains_of_contains(s.range, box_pair(n.suc), d - 1)
                        map(s.range, box_pair(n.suc)).contains(box_pair(n.suc, d - 1))
                        box_pair(n.suc, d - 1) = Pair.new(n.suc, shift_one(d - 1))
                        shift_one(d - 1) = d
                        box_pair(n.suc, d - 1) = Pair.new(n.suc, d)
                        c = n.suc
                        box_pair(n.suc, d - 1) = Pair.new(c, d)
                        map(s.range, box_pair(n.suc)).contains(Pair.new(c, d))
                        add_contains_right(pair_boxes(n, s), map(s.range, box_pair(n.suc)), Pair.new(c, d))
                        (pair_boxes(n, s) + map(s.range, box_pair(n.suc))).contains(Pair.new(c, d))
                        pair_boxes(n.suc, s) = pair_boxes(n, s) + map(s.range, box_pair(n.suc))
                        pair_boxes(n.suc, s).contains(Pair.new(c, d))
                    }
                    if c != n.suc {
                        lt_or_lte(c, n.suc)
                        if c < n.suc {
                            lt_imp_lte_suc(c, n.suc)
                            c.suc <= n.suc
                            lte_cancel_suc(c, n)
                            c <= n
                            p(n)
                            (Nat.1 <= c and c <= n and Nat.1 <= d and b <= s implies pair_boxes(n, s).contains(Pair.new(c, d)))
                            Nat.1 <= c and c <= n and Nat.1 <= d and b <= s
                            pair_boxes(n, s).contains(Pair.new(c, d))
                            add_contains_left(pair_boxes(n, s), map(s.range, box_pair(n.suc)), Pair.new(c, d))
                            (pair_boxes(n, s) + map(s.range, box_pair(n.suc))).contains(Pair.new(c, d))
                            pair_boxes(n.suc, s) = pair_boxes(n, s) + map(s.range, box_pair(n.suc))
                            pair_boxes(n.suc, s).contains(Pair.new(c, d))
                        }
                        if n.suc <= c {
                            c != n.suc
                            n.suc < c
                            lt_and_lte(n.suc, c, n.suc)
                            n.suc < n.suc
                            lt_not_ref(n.suc)
                            false
                        }
                        pair_boxes(n.suc, s).contains(Pair.new(c, d))
                    }
                    pair_boxes(n.suc, s).contains(Pair.new(c, d))
                }
            }
            p(n.suc)
        }
    }
    p(r)
}

/// The unique image list is no longer than any list containing the image.
theorem map_unique_length_le_of_contains[T, U](items: List[T], targets: List[U], f: T -> U) {
    forall(x: T) {
        items.contains(x) implies targets.contains(f(x))
    } implies map(items, f).unique.length <= targets.length
} by {
    if forall(x: T) {
        items.contains(x) implies targets.contains(f(x))
    } {
        forall(y: U) {
            if map(items, f).contains(y) {
                map_contains(items, f, y)
                let (x: T) satisfy {
                    items.contains(x) and f(x) = y
                }
                targets.contains(y)
            }
        }
        unique_is_smallest_containing_list(map(items, f), targets)
        map(items, f).unique.length <= targets.length
    }
}

/// The pigeonhole principle, naming the members of the source list that collide.
theorem pigeonhole_collision_in[T, U](items: List[T], targets: List[U], f: T -> U) {
    items.is_unique and items.length > targets.length and
    forall(x: T) {
        items.contains(x) implies targets.contains(f(x))
    } implies exists(x: T, y: T) {
        items.contains(x) and items.contains(y) and x != y and f(x) = f(y)
    }
} by {
    if items.is_unique and items.length > targets.length and forall(x: T) {
        items.contains(x) implies targets.contains(f(x))
    } {
        map_unique_length_le_of_contains(items, targets, f)
        map(items, f).unique.length <= targets.length
        targets.length < items.length
        lte_and_lt(map(items, f).unique.length, targets.length, items.length)
        map(items, f).unique.length < items.length
        if map(items, f).is_unique {
            map(items, f).unique.length = map(items, f).length
            map(items, f).length = items.length
            map(items, f).unique.length = items.length
            false
        }
        not map(items, f).is_unique
        pigeonhole_unique_map_in(items, f)
        exists(x: T, y: T) {
            items.contains(x) and items.contains(y) and x != y and f(x) = f(y)
        }
    }
}

/// The pigeonhole step of the Erdős–Szekeres proof.
theorem esz_pigeonhole[T: LinearOrder](r: Nat, s: Nat, xs: List[T]) {
    xs.is_unique and xs.length > r * s and
    not chain_geq(xs, lt_at(xs), r.suc) and
    not chain_geq(xs, gt_at(xs), s.suc)
    implies exists(i: Nat, j: Nat) {
        i < j and j < xs.length and esz_pair(xs, i) = esz_pair(xs, j)
    }
} by {
    if xs.is_unique and xs.length > r * s and
        not chain_geq(xs, lt_at(xs), r.suc) and
        not chain_geq(xs, gt_at(xs), s.suc) {
        forall(idx: Nat) {
            if xs.length.range.contains(idx) {
                lt_of_range_contains(xs.length, idx)
                idx < xs.length
                chain_val_geq_one(xs, lt_at(xs), idx)
                Nat.1 <= chain_val(xs, lt_at(xs), idx)
                chain_val_bound(xs, lt_at(xs), idx, r)
                idx < xs.length and not chain_geq(xs, lt_at(xs), r.suc)
                chain_val(xs, lt_at(xs), idx) <= r
                chain_val_geq_one(xs, gt_at(xs), idx)
                Nat.1 <= chain_val(xs, gt_at(xs), idx)
                chain_val_bound(xs, gt_at(xs), idx, s)
                chain_val(xs, gt_at(xs), idx) <= s
                pair_boxes_contains(r, s, chain_val(xs, lt_at(xs), idx), chain_val(xs, gt_at(xs), idx))
                pair_boxes(r, s).contains(Pair.new(chain_val(xs, lt_at(xs), idx), chain_val(xs, gt_at(xs), idx)))
                esz_pair(xs, idx) = Pair.new(chain_val(xs, lt_at(xs), idx), chain_val(xs, gt_at(xs), idx))
                pair_boxes(r, s).contains(esz_pair(xs, idx))
            }
            xs.length.range.contains(idx) implies pair_boxes(r, s).contains(esz_pair(xs, idx))
        }
        map_unique_length_le_of_contains(xs.length.range, pair_boxes(r, s), esz_pair_fn(xs))
        map(xs.length.range, esz_pair_fn(xs)).unique.length <= pair_boxes(r, s).length
        pair_boxes_length(r, s)
        pair_boxes(r, s).length = r * s
        map(xs.length.range, esz_pair_fn(xs)).unique.length <= r * s
        r * s < xs.length
        lte_and_lt(map(xs.length.range, esz_pair_fn(xs)).unique.length, r * s, xs.length)
        map(xs.length.range, esz_pair_fn(xs)).unique.length < xs.length
        if map(xs.length.range, esz_pair_fn(xs)).is_unique {
            map(xs.length.range, esz_pair_fn(xs)).unique.length = map(xs.length.range, esz_pair_fn(xs)).length
            map(xs.length.range, esz_pair_fn(xs)).length = xs.length.range.length
            xs.length.range.length = xs.length
            map(xs.length.range, esz_pair_fn(xs)).unique.length = xs.length
            false
        }
        not map(xs.length.range, esz_pair_fn(xs)).is_unique
        range_is_unique(xs.length)
        pigeonhole_unique_map_in(xs.length.range, esz_pair_fn(xs))
        exists(x: Nat, y: Nat) {
            xs.length.range.contains(x) and xs.length.range.contains(y) and x != y and esz_pair(xs, x) = esz_pair(xs, y)
        }
        let (x: Nat, y: Nat) satisfy {
            xs.length.range.contains(x) and xs.length.range.contains(y) and x != y and esz_pair(xs, x) = esz_pair(xs, y)
        }
        lt_of_range_contains(xs.length, x)
        lt_of_range_contains(xs.length, y)
        x < xs.length
        y < xs.length
        trichotomy(x, y)
        if x < y {
            x < y and y < xs.length and esz_pair(xs, x) = esz_pair(xs, y)
            exists(i: Nat, j: Nat) {
                i < j and j < xs.length and esz_pair(xs, i) = esz_pair(xs, j)
            }
        }
        if y < x {
            y < x and x < xs.length and esz_pair(xs, y) = esz_pair(xs, x)
            exists(i: Nat, j: Nat) {
                i < j and j < xs.length and esz_pair(xs, i) = esz_pair(xs, j)
            }
        }
        exists(i: Nat, j: Nat) {
            i < j and j < xs.length and esz_pair(xs, i) = esz_pair(xs, j)
        }
    }
}

/// The Erdős–Szekeres theorem: more than `r * s` distinct elements of a linear order
/// contain an increasing subsequence of length `r + 1` or a decreasing subsequence
/// of length `s + 1`.
theorem erdos_szekeres[T: LinearOrder](r: Nat, s: Nat, xs: List[T]) {
    xs.is_unique and xs.length > r * s implies
    chain_geq(xs, lt_at(xs), r.suc) or chain_geq(xs, gt_at(xs), s.suc)
} by {
    if xs.is_unique and xs.length > r * s {
        if not (chain_geq(xs, lt_at(xs), r.suc) or chain_geq(xs, gt_at(xs), s.suc)) {
            not chain_geq(xs, lt_at(xs), r.suc)
            not chain_geq(xs, gt_at(xs), s.suc)
            esz_pigeonhole(r, s, xs)
            exists(i: Nat, j: Nat) {
                i < j and j < xs.length and esz_pair(xs, i) = esz_pair(xs, j)
            }
            let (i: Nat, j: Nat) satisfy {
                i < j and j < xs.length and esz_pair(xs, i) = esz_pair(xs, j)
            }
            esz_pair_distinct(xs, i, j)
            xs.is_unique and i < j and j < xs.length
            esz_pair(xs, i) != esz_pair(xs, j)
            false
        }
        chain_geq(xs, lt_at(xs), r.suc) or chain_geq(xs, gt_at(xs), s.suc)
    }
}
