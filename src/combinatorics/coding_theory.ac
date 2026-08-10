/// Coding theory foundations: binary words, Hamming distance, error-correcting
/// codes, and the sphere-packing (Hamming) bound.
///
/// A binary word of length `n` is a list of `n` booleans.  The Hamming
/// distance between two words of the same length is the number of positions
/// at which they differ, and a code is a finite set of words of a common
/// length.  Hamming distance is a metric on the words of a fixed length:
/// non-negativity, symmetry, and the triangle inequality are proved below.
///
/// The sphere-packing (Hamming) bound says that a code of length `n` with
/// minimum distance `d` and size `M` satisfies
///
///     M * sum_{i = 0}^{floor((d - 1) / 2)} C(n, i) <= 2^n ,
///
/// because the balls of radius `floor((d - 1) / 2)` around the codewords are
/// disjoint subsets of the `2^n` words of length `n`.  The general statement
/// requires a counting argument for balls; the bound is proved here for the
/// small case `n = 3`, `d = 3`, where it reads `M * (1 + 3) <= 8`, so
/// `M <= 2`: the repetition code `{000, 111}` is the largest such code, and
/// it corrects one error.

from nat import Nat
from list import List
from nat import lte_ref, lte_trans, lte_antisymm, lte_cancel_suc, lt_imp_lte_suc, lt_imp_lt_suc,
    lt_suc, lte_suc_suc, alt_suc_ne_zero, lte_add_left, lte_add_right,
    only_zero_lte_zero, lte_imp_not_lt, add_to_zero, lt_and_lte
from nat import trichotomy, lte_mul_both, lt_mul_both, lt_or_lte, lt_diff,
    zero_or_suc, mul_cancel_left, lt_not_ref
from nat import pow_add
from combinatorics import binom, choose_zero, choose_one

numerals Nat

/// The Hamming distance between two binary words: the number of positions at
/// which the two words differ.  The distance is meaningful for words of the
/// same length; for words of unequal length only the positions common to both
/// words are compared.
define hamming(xs: List[Bool], ys: List[Bool]) -> Nat {
    match xs {
        List.nil {
            match ys {
                List.nil {
                    Nat.0
                }
                List.cons(hy, ty) {
                    Nat.0
                }
            }
        }
        List.cons(hx, tx) {
            match ys {
                List.nil {
                    Nat.0
                }
                List.cons(hy, ty) {
                    if hx != hy {
                        Nat.1 + hamming(tx, ty)
                    } else {
                        hamming(tx, ty)
                    }
                }
            }
        }
    }
}

/// One when two bits differ, zero when they agree.
define bit_diff(a: Bool, b: Bool) -> Nat {
    if a != b {
        Nat.1
    } else {
        Nat.0
    }
}

/// A list of length zero is the empty list.
theorem length_zero_imp_nil[T](list: List[T]) {
    list.length = Nat.0 implies list = List.nil[T]
} by {
    match list {
        List.nil {
            if list.length = Nat.0 {
                list = List.nil[T]
            }
        }
        List.cons(head, tail) {
            tail.length.suc != Nat.0
            if list.length = Nat.0 {
                false
            }
        }
    }
}

/// The Hamming distance of a consed pair of words is the bit difference of
/// the heads plus the Hamming distance of the tails.
theorem hamming_cons_eq(xs: List[Bool], ys: List[Bool], hx: Bool, hy: Bool) {
    hamming(List.cons(hx, xs), List.cons(hy, ys)) = bit_diff(hx, hy) + hamming(xs, ys)
} by {
    if hx != hy {
        hamming(List.cons(hx, xs), List.cons(hy, ys)) = Nat.1 + hamming(xs, ys)
        bit_diff(hx, hy) = Nat.1
        Nat.1 + hamming(xs, ys) = bit_diff(hx, hy) + hamming(xs, ys)
        hamming(List.cons(hx, xs), List.cons(hy, ys)) = bit_diff(hx, hy) + hamming(xs, ys)
    }
    if not (hx != hy) {
        hamming(List.cons(hx, xs), List.cons(hy, ys)) = hamming(xs, ys)
        bit_diff(hx, hy) = Nat.0
        hamming(xs, ys) = bit_diff(hx, hy) + hamming(xs, ys)
        hamming(List.cons(hx, xs), List.cons(hy, ys)) = bit_diff(hx, hy) + hamming(xs, ys)
    }
}

/// The bit difference is symmetric.
theorem bit_diff_sym(hx: Bool, hy: Bool) {
    bit_diff(hx, hy) = bit_diff(hy, hx)
} by {
    if hx != hy {
        bit_diff(hx, hy) = Nat.1
        if hy != hx {
            bit_diff(hy, hx) = Nat.1
            bit_diff(hx, hy) = bit_diff(hy, hx)
        }
        if not (hy != hx) {
            hy = hx
            hx != hx
            false
        }
    }
    if not (hx != hy) {
        bit_diff(hx, hy) = Nat.0
        if hy != hx {
            hy != hy
            false
        }
        if not (hy != hx) {
            bit_diff(hy, hx) = Nat.0
            bit_diff(hx, hy) = bit_diff(hy, hx)
        }
    }
}

/// Adding an inequality to both sides of another inequality preserves the
/// ordering.
theorem lte_add_both(a: Nat, b: Nat, c: Nat, d: Nat) {
    a <= b and c <= d implies a + c <= b + d
} by {
    if a <= b and c <= d {
        lte_add_left(c, a, b)
        c + a <= c + b
        lte_add_right(b, c, d)
        c + b <= d + b
        lte_trans(c + a, c + b, d + b)
        c + a <= d + b
        a + c = c + a
        b + d = d + b
        a + c <= b + d
    }
}

/// A natural number is at most one plus itself.
theorem n_lte_one_plus_n(n: Nat) {
    n <= Nat.1 + n
} by {
    define p(m: Nat) -> Bool {
        m <= Nat.1 + m
    }
    Nat.1 + Nat.0 = Nat.1
    p(Nat.0)
    forall(k: Nat) {
        if p(k) {
            lte_suc_suc(k, Nat.1 + k)
            k.suc <= (Nat.1 + k).suc
            Nat.1 + k.suc = (Nat.1 + k).suc
            k.suc <= Nat.1 + k.suc
            p(k.suc)
        }
    }
    p(n)
}

/// Zero is at most every natural number.
theorem zero_lte(n: Nat) {
    Nat.0 <= n
} by {
    define p(m: Nat) -> Bool {
        Nat.0 <= m
    }
    p(Nat.0)
    forall(k: Nat) {
        if p(k) {
            n_lte_one_plus_n(k)
            k <= Nat.1 + k
            lte_trans(Nat.0, k, Nat.1 + k)
            Nat.0 <= Nat.1 + k
            p(k.suc)
        }
    }
    p(n)
}

/// If two bits differ then at least one of the two intermediate comparisons
/// also differs: for booleans, `hx != hz` implies `hx != hy` or `hy != hz`.
theorem bool_diff_or(hx: Bool, hy: Bool, hz: Bool) {
    hx != hz implies hx != hy or hy != hz
} by {
    if hx != hz {
        if hx {
            if hy {
                if hz {
                    false
                }
                if not hz {
                    hy != hz
                    hx != hy or hy != hz
                }
            }
            if not hy {
                hx != hy
                hx != hy or hy != hz
            }
        }
        if not hx {
            if hy {
                hx != hy
                hx != hy or hy != hz
            }
            if not hy {
                if hz {
                    hy != hz
                    hx != hy or hy != hz
                }
                if not hz {
                    false
                }
            }
        }
    }
}

/// The bit difference is subadditive: the difference of the endpoints is at
/// most the sum of the differences through any intermediate bit.
theorem bit_diff_le_sum(hx: Bool, hy: Bool, hz: Bool) {
    bit_diff(hx, hz) <= bit_diff(hx, hy) + bit_diff(hy, hz)
} by {
    if hx != hz {
        bool_diff_or(hx, hy, hz)
        if hx != hy or hy != hz {
            if hx != hy {
                bit_diff(hx, hz) = Nat.1
                bit_diff(hx, hy) = Nat.1
                zero_lte(bit_diff(hy, hz))
                lte_add_left(Nat.1, Nat.0, bit_diff(hy, hz))
                Nat.1 + Nat.0 <= Nat.1 + bit_diff(hy, hz)
                Nat.1 <= Nat.1 + bit_diff(hy, hz)
                bit_diff(hx, hz) <= bit_diff(hx, hy) + bit_diff(hy, hz)
            } else {
                hy != hz
                bit_diff(hx, hz) = Nat.1
                bit_diff(hy, hz) = Nat.1
                zero_lte(bit_diff(hx, hy))
                lte_add_right(Nat.1, Nat.0, bit_diff(hx, hy))
                Nat.0 + Nat.1 <= bit_diff(hx, hy) + Nat.1
                Nat.1 <= bit_diff(hx, hy) + Nat.1
                Nat.1 <= bit_diff(hx, hy) + bit_diff(hy, hz)
                bit_diff(hx, hz) <= bit_diff(hx, hy) + bit_diff(hy, hz)
            }
        }
        bit_diff(hx, hz) <= bit_diff(hx, hy) + bit_diff(hy, hz)
    }
    if not (hx != hz) {
        bit_diff(hx, hz) = Nat.0
        zero_lte(bit_diff(hx, hy) + bit_diff(hy, hz))
        bit_diff(hx, hz) <= bit_diff(hx, hy) + bit_diff(hy, hz)
    }
}

/// Hamming distance is non-negative.
theorem hamming_nonneg(xs: List[Bool], ys: List[Bool]) {
    Nat.0 <= hamming(xs, ys)
}

/// Hamming distance between a word and itself is zero.
theorem hamming_self(xs: List[Bool]) {
    hamming(xs, xs) = Nat.0
} by {
    define p(ws: List[Bool]) -> Bool {
        hamming(ws, ws) = Nat.0
    }
    hamming(List.nil[Bool], List.nil[Bool]) = Nat.0
    p(List.nil[Bool])
    forall(hx: Bool, tx: List[Bool]) {
        if p(tx) {
            hamming(List.cons(hx, tx), List.cons(hx, tx)) = hamming(tx, tx)
            p(List.cons(hx, tx))
        }
    }
    p(xs)
}

/// Hamming distance is symmetric for words of the same length.
theorem hamming_sym(xs: List[Bool], ys: List[Bool]) {
    xs.length = ys.length implies hamming(xs, ys) = hamming(ys, xs)
} by {
    define p(us: List[Bool]) -> Bool {
        forall(vs: List[Bool]) {
            us.length = vs.length implies hamming(us, vs) = hamming(vs, us)
        }
    }
    p(List.nil[Bool]) = forall(vs: List[Bool]) {
        List.nil[Bool].length = vs.length implies hamming(List.nil[Bool], vs) = hamming(vs, List.nil[Bool])
    }
    if not p(List.nil[Bool]) {
        let vs0: List[Bool] satisfy {
            List.nil[Bool].length = vs0.length and not (hamming(List.nil[Bool], vs0) = hamming(vs0, List.nil[Bool]))
        }
        List.nil[Bool].length = Nat.0
        vs0.length = Nat.0
        length_zero_imp_nil[Bool](vs0)
        vs0 = List.nil[Bool]
        hamming(List.nil[Bool], vs0) = hamming(vs0, List.nil[Bool])
        false
    }
    p(List.nil[Bool])
    forall(hx: Bool, tx: List[Bool]) {
        if p(tx) {
            forall(vs: List[Bool]) {
                if List.cons(hx, tx).length = vs.length {
                    List.cons(hx, tx).length = tx.length.suc
                    tx.length.suc = vs.length
                    match vs {
                        List.nil {
                            List.nil[Bool].length = Nat.0
                            List.cons(hx, tx).length = Nat.0
                            tx.length.suc = Nat.0
                            alt_suc_ne_zero(tx.length)
                            tx.length.suc != Nat.0
                            false
                        }
                        List.cons(hy, ty) {
                            List.cons(hy, ty).length = ty.length.suc
                            tx.length.suc = ty.length.suc
                            tx.length.suc <= ty.length.suc
                            lte_cancel_suc(tx.length, ty.length)
                            tx.length <= ty.length
                            ty.length.suc <= tx.length.suc
                            lte_cancel_suc(ty.length, tx.length)
                            ty.length <= tx.length
                            lte_antisymm(tx.length, ty.length)
                            tx.length = ty.length
                            p(tx)
                            hamming(tx, ty) = hamming(ty, tx)
                            hamming_cons_eq(tx, ty, hx, hy)
                            hamming(List.cons(hx, tx), List.cons(hy, ty)) = bit_diff(hx, hy) + hamming(tx, ty)
                            hamming_cons_eq(ty, tx, hy, hx)
                            hamming(List.cons(hy, ty), List.cons(hx, tx)) = bit_diff(hy, hx) + hamming(ty, tx)
                            bit_diff_sym(hx, hy)
                            bit_diff(hx, hy) = bit_diff(hy, hx)
                            bit_diff(hx, hy) + hamming(tx, ty) = bit_diff(hy, hx) + hamming(ty, tx)
                            hamming(List.cons(hx, tx), List.cons(hy, ty)) = hamming(List.cons(hy, ty), List.cons(hx, tx))
                        }
                    }
                }
            }
            p(List.cons(hx, tx)) = forall(vs: List[Bool]) {
                List.cons(hx, tx).length = vs.length implies hamming(List.cons(hx, tx), vs) = hamming(vs, List.cons(hx, tx))
            }
            if not p(List.cons(hx, tx)) {
                let vs0: List[Bool] satisfy {
                    List.cons(hx, tx).length = vs0.length and not (hamming(List.cons(hx, tx), vs0) = hamming(vs0, List.cons(hx, tx)))
                }
                match vs0 {
                    List.nil {
                        List.nil[Bool].length = Nat.0
                        List.cons(hx, tx).length = Nat.0
                        tx.length.suc = Nat.0
                        alt_suc_ne_zero(tx.length)
                        tx.length.suc != Nat.0
                        false
                    }
                    List.cons(hy, ty) {
                        List.cons(hx, tx).length = tx.length.suc
                        List.cons(hy, ty).length = ty.length.suc
                        tx.length.suc = ty.length.suc
                        tx.length.suc <= ty.length.suc
                        lte_cancel_suc(tx.length, ty.length)
                        tx.length <= ty.length
                        ty.length.suc <= tx.length.suc
                        lte_cancel_suc(ty.length, tx.length)
                        ty.length <= tx.length
                        lte_antisymm(tx.length, ty.length)
                        tx.length = ty.length
                        p(tx)
                        hamming(tx, ty) = hamming(ty, tx)
                        hamming_cons_eq(tx, ty, hx, hy)
                        hamming(List.cons(hx, tx), List.cons(hy, ty)) = bit_diff(hx, hy) + hamming(tx, ty)
                        hamming_cons_eq(ty, tx, hy, hx)
                        hamming(List.cons(hy, ty), List.cons(hx, tx)) = bit_diff(hy, hx) + hamming(ty, tx)
                        bit_diff_sym(hx, hy)
                        bit_diff(hx, hy) = bit_diff(hy, hx)
                        bit_diff(hx, hy) + hamming(tx, ty) = bit_diff(hy, hx) + hamming(ty, tx)
                        hamming(List.cons(hx, tx), List.cons(hy, ty)) = hamming(List.cons(hy, ty), List.cons(hx, tx))
                        hamming(List.cons(hx, tx), vs0) = hamming(vs0, List.cons(hx, tx))
                        false
                    }
                }
            }
            p(List.cons(hx, tx))
        }
    }
    p(xs)
}

/// If the bit difference is zero then the bits are equal.
theorem bit_diff_zero_imp_eq(hx: Bool, hy: Bool) {
    bit_diff(hx, hy) = Nat.0 implies hx = hy
} by {
    if bit_diff(hx, hy) = Nat.0 {
        if hx != hy {
            bit_diff(hx, hy) = Nat.1
            Nat.1 = Nat.0
            false
        }
        hx = hy
    }
}

/// Hamming distance satisfies the triangle inequality for words of the same
/// length: the distance from `xs` to `zs` is no larger than the distance via
/// any intermediate word `ys`.
theorem hamming_triangle(xs: List[Bool], ys: List[Bool], zs: List[Bool]) {
    xs.length = ys.length and ys.length = zs.length implies
        hamming(xs, zs) <= hamming(xs, ys) + hamming(ys, zs)
} by {
    define p(n: Nat) -> Bool {
        forall(us: List[Bool], vs: List[Bool], ws: List[Bool]) {
            us.length = n and vs.length = n and ws.length = n implies
                hamming(us, ws) <= hamming(us, vs) + hamming(vs, ws)
        }
    }
    p(Nat.0) = forall(us: List[Bool], vs: List[Bool], ws: List[Bool]) {
        us.length = Nat.0 and vs.length = Nat.0 and ws.length = Nat.0 implies
            hamming(us, ws) <= hamming(us, vs) + hamming(vs, ws)
    }
    if not p(Nat.0) {
        let (us0: List[Bool], vs0: List[Bool], ws0: List[Bool]) satisfy {
            us0.length = Nat.0 and vs0.length = Nat.0 and ws0.length = Nat.0 and
                not (hamming(us0, ws0) <= hamming(us0, vs0) + hamming(vs0, ws0))
        }
        us0.length = Nat.0
        length_zero_imp_nil[Bool](us0)
        us0 = List.nil[Bool]
        vs0.length = Nat.0
        length_zero_imp_nil[Bool](vs0)
        vs0 = List.nil[Bool]
        ws0.length = Nat.0
        length_zero_imp_nil[Bool](ws0)
        ws0 = List.nil[Bool]
        hamming(us0, ws0) = hamming(List.nil[Bool], List.nil[Bool])
        hamming(List.nil[Bool], List.nil[Bool]) = Nat.0
        hamming(us0, vs0) = hamming(List.nil[Bool], List.nil[Bool])
        hamming(vs0, ws0) = hamming(List.nil[Bool], List.nil[Bool])
        Nat.0 + Nat.0 = Nat.0
        Nat.0 <= Nat.0 + Nat.0
        hamming(us0, ws0) <= hamming(us0, vs0) + hamming(vs0, ws0)
        false
    }
    p(Nat.0)
    forall(n: Nat) {
        if p(n) {
            forall(us: List[Bool], vs: List[Bool], ws: List[Bool]) {
                if us.length = n.suc and vs.length = n.suc and ws.length = n.suc {
                    match us {
                        List.nil {
                            us.length = Nat.0
                            Nat.0 = n.suc
                            false
                        }
                        List.cons(hx, tx) {
                            List.cons(hx, tx).length = tx.length.suc
                            tx.length.suc = n.suc
                            tx.length.suc <= n.suc
                            lte_cancel_suc(tx.length, n)
                            tx.length <= n
                            n.suc <= tx.length.suc
                            lte_cancel_suc(n, tx.length)
                            n <= tx.length
                            lte_antisymm(tx.length, n)
                            tx.length = n
                            match vs {
                                List.nil {
                                    vs.length = Nat.0
                                    Nat.0 = n.suc
                                    false
                                }
                                List.cons(hy, ty) {
                                    List.cons(hy, ty).length = ty.length.suc
                                    ty.length.suc = n.suc
                                    ty.length.suc <= n.suc
                                    lte_cancel_suc(ty.length, n)
                                    ty.length <= n
                                    n.suc <= ty.length.suc
                                    lte_cancel_suc(n, ty.length)
                                    n <= ty.length
                                    lte_antisymm(ty.length, n)
                                    ty.length = n
                                    match ws {
                                        List.nil {
                                            ws.length = Nat.0
                                            Nat.0 = n.suc
                                            false
                                        }
                                        List.cons(hz, tz) {
                                            List.cons(hz, tz).length = tz.length.suc
                                            tz.length.suc = n.suc
                                            tz.length.suc <= n.suc
                                            lte_cancel_suc(tz.length, n)
                                            tz.length <= n
                                            n.suc <= tz.length.suc
                                            lte_cancel_suc(n, tz.length)
                                            n <= tz.length
                                            lte_antisymm(tz.length, n)
                                            tz.length = n
                                            p(n)
                                            tx.length = n and ty.length = n and tz.length = n
                                            hamming(tx, tz) <= hamming(tx, ty) + hamming(ty, tz)
                                            bit_diff_le_sum(hx, hy, hz)
                                            bit_diff(hx, hz) <= bit_diff(hx, hy) + bit_diff(hy, hz)
                                            lte_add_both(bit_diff(hx, hz), bit_diff(hx, hy) + bit_diff(hy, hz), hamming(tx, tz), hamming(tx, ty) + hamming(ty, tz))
                                            bit_diff(hx, hz) + hamming(tx, tz) <= (bit_diff(hx, hy) + bit_diff(hy, hz)) + (hamming(tx, ty) + hamming(ty, tz))
                                            (bit_diff(hx, hy) + bit_diff(hy, hz)) + (hamming(tx, ty) + hamming(ty, tz)) = bit_diff(hx, hy) + hamming(tx, ty) + bit_diff(hy, hz) + hamming(ty, tz)
                                            bit_diff(hx, hz) + hamming(tx, tz) <= bit_diff(hx, hy) + hamming(tx, ty) + bit_diff(hy, hz) + hamming(ty, tz)
                                            hamming_cons_eq(tx, tz, hx, hz)
                                            hamming(List.cons(hx, tx), List.cons(hz, tz)) = bit_diff(hx, hz) + hamming(tx, tz)
                                            hamming_cons_eq(tx, ty, hx, hy)
                                            hamming(List.cons(hx, tx), List.cons(hy, ty)) = bit_diff(hx, hy) + hamming(tx, ty)
                                            hamming_cons_eq(ty, tz, hy, hz)
                                            hamming(List.cons(hy, ty), List.cons(hz, tz)) = bit_diff(hy, hz) + hamming(ty, tz)
                                            hamming(List.cons(hx, tx), List.cons(hz, tz)) <= hamming(List.cons(hx, tx), List.cons(hy, ty)) + hamming(List.cons(hy, ty), List.cons(hz, tz))
                                            hamming(us, ws) <= hamming(us, vs) + hamming(vs, ws)
                                        }
                                    }
                                }
                            }
                        }
                    }
                }
            }
            p(n.suc) = forall(us: List[Bool], vs: List[Bool], ws: List[Bool]) {
                us.length = n.suc and vs.length = n.suc and ws.length = n.suc implies
                    hamming(us, ws) <= hamming(us, vs) + hamming(vs, ws)
            }
            p(n.suc)
        }
    }
    p(xs.length)
    ys.length = xs.length
    zs.length = xs.length
    hamming(xs, zs) <= hamming(xs, ys) + hamming(ys, zs)
}

/// A word at Hamming distance zero from another word of the same length is
/// equal to it.
theorem hamming_zero_imp_eq(xs: List[Bool], ys: List[Bool]) {
    xs.length = ys.length and hamming(xs, ys) = Nat.0 implies xs = ys
} by {
    define p(n: Nat) -> Bool {
        forall(us: List[Bool], vs: List[Bool]) {
            us.length = n and vs.length = n and hamming(us, vs) = Nat.0 implies us = vs
        }
    }
    p(Nat.0) = forall(us: List[Bool], vs: List[Bool]) {
        us.length = Nat.0 and vs.length = Nat.0 and hamming(us, vs) = Nat.0 implies us = vs
    }
    if not p(Nat.0) {
        let (us0: List[Bool], vs0: List[Bool]) satisfy {
            us0.length = Nat.0 and vs0.length = Nat.0 and hamming(us0, vs0) = Nat.0 and
                not (us0 = vs0)
        }
        us0.length = Nat.0
        length_zero_imp_nil[Bool](us0)
        us0 = List.nil[Bool]
        vs0.length = Nat.0
        length_zero_imp_nil[Bool](vs0)
        vs0 = List.nil[Bool]
        us0 = vs0
        false
    }
    p(Nat.0)
    forall(n: Nat) {
        if p(n) {
            forall(us: List[Bool], vs: List[Bool]) {
                if us.length = n.suc and vs.length = n.suc and hamming(us, vs) = Nat.0 {
                    match us {
                        List.nil {
                            us.length = Nat.0
                            Nat.0 = n.suc
                            false
                        }
                        List.cons(hx, tx) {
                            List.cons(hx, tx).length = tx.length.suc
                            tx.length.suc = n.suc
                            tx.length.suc <= n.suc
                            lte_cancel_suc(tx.length, n)
                            tx.length <= n
                            n.suc <= tx.length.suc
                            lte_cancel_suc(n, tx.length)
                            n <= tx.length
                            lte_antisymm(tx.length, n)
                            tx.length = n
                            match vs {
                                List.nil {
                                    vs.length = Nat.0
                                    Nat.0 = n.suc
                                    false
                                }
                                List.cons(hy, ty) {
                                    List.cons(hy, ty).length = ty.length.suc
                                    ty.length.suc = n.suc
                                    ty.length.suc <= n.suc
                                    lte_cancel_suc(ty.length, n)
                                    ty.length <= n
                                    n.suc <= ty.length.suc
                                    lte_cancel_suc(n, ty.length)
                                    n <= ty.length
                                    lte_antisymm(ty.length, n)
                                    ty.length = n
                                    hamming_cons_eq(tx, ty, hx, hy)
                                    hamming(List.cons(hx, tx), List.cons(hy, ty)) = bit_diff(hx, hy) + hamming(tx, ty)
                                    hamming(List.cons(hx, tx), List.cons(hy, ty)) = Nat.0
                                    bit_diff(hx, hy) + hamming(tx, ty) = Nat.0
                                    add_to_zero(bit_diff(hx, hy), hamming(tx, ty))
                                    bit_diff(hx, hy) = Nat.0
                                    hamming(tx, ty) = Nat.0
                                    bit_diff_zero_imp_eq(hx, hy)
                                    hx = hy
                                    p(n)
                                    tx.length = n and ty.length = n and hamming(tx, ty) = Nat.0
                                    tx = ty
                                    List.cons(hx, tx) = List.cons(hy, ty)
                                    us = vs
                                }
                            }
                        }
                    }
                }
            }
            p(n.suc) = forall(us: List[Bool], vs: List[Bool]) {
                us.length = n.suc and vs.length = n.suc and hamming(us, vs) = Nat.0 implies us = vs
            }
            p(n.suc)
        }
    }
    p(xs.length)
    ys.length = xs.length
    hamming(xs, ys) = Nat.0
    xs = ys
}

/// Two words of the same length are equal exactly when their Hamming distance
/// is zero.
theorem hamming_zero_iff_eq(xs: List[Bool], ys: List[Bool]) {
    xs.length = ys.length implies ((hamming(xs, ys) = Nat.0) = (xs = ys))
} by {
    if xs.length = ys.length {
        hamming_zero_imp_eq(xs, ys)
        if hamming(xs, ys) = Nat.0 {
            xs = ys
        }
        if xs = ys {
            hamming_self(xs)
            hamming(xs, ys) = Nat.0
        }
        (hamming(xs, ys) = Nat.0) = (xs = ys)
    }
}
/// True if a word has the given length.
define is_word_length(xs: List[Bool], n: Nat) -> Bool {
    xs.length = n
}

/// True if a list of words is a code of length `n`: the words are distinct
/// and every word has length `n`.
define is_code(words: List[List[Bool]], n: Nat) -> Bool {
    words.is_unique and forall(w: List[Bool]) {
        words.contains(w) implies w.length = n
    }
}

/// True if every two distinct words of the code are at Hamming distance at
/// least `d` apart.
define min_distance_at_least(words: List[List[Bool]], d: Nat) -> Bool {
    forall(x: List[Bool], y: List[Bool]) {
        words.contains(x) and words.contains(y) and x != y implies d <= hamming(x, y)
    }
}

/// The all-zero word of a given length.
define zero_word(n: Nat) -> List[Bool] {
    match n {
        Nat.zero {
            List.nil[Bool]
        }
        Nat.suc(k) {
            List.cons(false, zero_word(k))
        }
    }
}

/// The all-one word of a given length.
define one_word(n: Nat) -> List[Bool] {
    match n {
        Nat.zero {
            List.nil[Bool]
        }
        Nat.suc(k) {
            List.cons(true, one_word(k))
        }
    }
}

/// The repetition code of length three: `{000, 111}`.
let repetition_code_three: List[List[Bool]] =
    List.cons(zero_word(3), List.cons(one_word(3), List.nil[List[Bool]]))

/// The word obtained from `000` by flipping the first bit.
let flip_first_three: List[Bool] =
    List.cons(true, zero_word(2))

/// The word obtained from `000` by flipping the second bit.
let flip_second_three: List[Bool] =
    List.cons(false, List.cons(true, zero_word(1)))

/// The word obtained from `000` by flipping the third bit.
let flip_third_three: List[Bool] =
    List.cons(false, List.cons(false, List.cons(true, zero_word(0))))


/// The zero word of length `n` has length `n`.
theorem zero_word_length(n: Nat) {
    zero_word(n).length = n
} by {
    define p(m: Nat) -> Bool {
        zero_word(m).length = m
    }
    zero_word(Nat.0) = List.nil[Bool]
    List.nil[Bool].length = Nat.0
    p(Nat.0)
    forall(k: Nat) {
        if p(k) {
            zero_word(k.suc) = List.cons(false, zero_word(k))
            List.cons(false, zero_word(k)).length = zero_word(k).length.suc
            zero_word(k).length = k
            zero_word(k.suc).length = k.suc
            p(k.suc)
        }
    }
    p(n)
}

/// The one word of length `n` has length `n`.
theorem one_word_length(n: Nat) {
    one_word(n).length = n
} by {
    define p(m: Nat) -> Bool {
        one_word(m).length = m
    }
    one_word(Nat.0) = List.nil[Bool]
    List.nil[Bool].length = Nat.0
    p(Nat.0)
    forall(k: Nat) {
        if p(k) {
            one_word(k.suc) = List.cons(true, one_word(k))
            List.cons(true, one_word(k)).length = one_word(k).length.suc
            one_word(k).length = k
            one_word(k.suc).length = k.suc
            p(k.suc)
        }
    }
    p(n)
}

/// The zero word and the one word of length `n` are distinct for positive `n`.
theorem zero_one_distinct(n: Nat) {
    0 < n implies zero_word(n) != one_word(n)
} by {
    if 0 < n {
        let k: Nat satisfy {
            k.suc = n
        }
        zero_word(n) = List.cons(false, zero_word(k))
        one_word(n) = List.cons(true, one_word(k))
        if zero_word(n) = one_word(n) {
            List.cons(false, zero_word(k)) = List.cons(true, one_word(k))
            false
        }
    }
}

/// The Hamming distance between the zero word and the one word of length `n`
/// is `n`.
theorem hamming_zero_one(n: Nat) {
    hamming(zero_word(n), one_word(n)) = n
} by {
    define p(m: Nat) -> Bool {
        hamming(zero_word(m), one_word(m)) = m
    }
    hamming(List.nil[Bool], List.nil[Bool]) = Nat.0
    p(Nat.0)
    forall(k: Nat) {
        if p(k) {
            zero_word(k.suc) = List.cons(false, zero_word(k))
            one_word(k.suc) = List.cons(true, one_word(k))
            hamming_cons_eq(zero_word(k), one_word(k), false, true)
            hamming(List.cons(false, zero_word(k)), List.cons(true, one_word(k))) = bit_diff(false, true) + hamming(zero_word(k), one_word(k))
            bit_diff(false, true) = Nat.1
            hamming(zero_word(k), one_word(k)) = k
            hamming(zero_word(k.suc), one_word(k.suc)) = k.suc
            p(k.suc)
        }
    }
    p(n)
}

/// Membership in a two-element list forces equality with one of its elements.
theorem contains_two_imp_eq[T](a: T, b: T, w: T) {
    List.cons(a, List.cons(b, List.nil[T])).contains(w) implies w = a or w = b
} by {
    if List.cons(a, List.cons(b, List.nil[T])).contains(w) {
        if w = a {
            w = a or w = b
        }
        if w != a {
            if w = b {
                w = a or w = b
            }
            if w != b {
                not List.cons(b, List.nil[T]).contains(w)
                false
            }
        }
    }
}

/// A two-element list of distinct elements is unique.
theorem two_unique_distinct[T](a: T, b: T) {
    a != b implies List.cons(a, List.cons(b, List.nil[T])).is_unique
} by {
    if a != b {
        List.cons(b, List.nil[T]).contains(a) = false
        List.nil[T].unique = List.nil[T]
        List.cons(b, List.nil[T]).unique = List.cons(b, List.nil[T])
        List.cons(a, List.cons(b, List.nil[T])).unique = List.cons(a, List.cons(b, List.nil[T]).unique)
        List.cons(a, List.cons(b, List.nil[T])).unique = List.cons(a, List.cons(b, List.nil[T]))
        List.cons(a, List.cons(b, List.nil[T])).is_unique
    }
}

/// Every word of the repetition code `{000, 111}` has length three.
theorem repetition_three_members_length(w: List[Bool]) {
    repetition_code_three.contains(w) implies w.length = 3
} by {
    if repetition_code_three.contains(w) {
        contains_two_imp_eq[List[Bool]](zero_word(3), one_word(3), w)
        w = zero_word(3) or w = one_word(3)
        if w = zero_word(3) or w = one_word(3) {
            if w = zero_word(3) {
                w.length = zero_word(3).length
                zero_word_length(3)
                zero_word(3).length = 3
                w.length = 3
            } else {
                w = one_word(3)
                w.length = one_word(3).length
                one_word_length(3)
                one_word(3).length = 3
                w.length = 3
            }
        }
        w.length = 3
    }
}

/// The repetition code `{000, 111}` is a code of length three.
theorem repetition_three_is_code {
    is_code(repetition_code_three, 3)
} by {
    zero_one_distinct(3)
    zero_word(3) != one_word(3)
    two_unique_distinct[List[Bool]](zero_word(3), one_word(3))
    repetition_code_three.is_unique
    forall(w: List[Bool]) { repetition_three_members_length(w) }
    is_code(repetition_code_three, 3) = (repetition_code_three.is_unique and forall(w: List[Bool]) {
        repetition_code_three.contains(w) implies w.length = 3
    })
    is_code(repetition_code_three, 3)
}

/// Any two distinct words of the repetition code `{000, 111}` are at Hamming
/// distance three.
theorem repetition_three_pair_distance(x: List[Bool], y: List[Bool]) {
    repetition_code_three.contains(x) and repetition_code_three.contains(y) and x != y implies 3 <= hamming(x, y)
} by {
    if repetition_code_three.contains(x) and repetition_code_three.contains(y) and x != y {
        contains_two_imp_eq[List[Bool]](zero_word(3), one_word(3), x)
        x = zero_word(3) or x = one_word(3)
        if x = zero_word(3) {
            contains_two_imp_eq[List[Bool]](zero_word(3), one_word(3), y)
            y = zero_word(3) or y = one_word(3)
            if y = zero_word(3) {
                x = y
                false
            }
            if not (y = zero_word(3)) {
                y = one_word(3)
                hamming(x, y) = hamming(zero_word(3), one_word(3))
                hamming_zero_one(3)
                hamming(zero_word(3), one_word(3)) = 3
                hamming(x, y) = 3
                3 <= hamming(x, y)
            }
            3 <= hamming(x, y)
        }
        if not (x = zero_word(3)) {
            x = one_word(3)
            contains_two_imp_eq[List[Bool]](zero_word(3), one_word(3), y)
            y = zero_word(3) or y = one_word(3)
            if y = zero_word(3) {
                zero_word_length(3)
                one_word_length(3)
                hamming_sym(one_word(3), zero_word(3))
                hamming(one_word(3), zero_word(3)) = hamming(zero_word(3), one_word(3))
                hamming_zero_one(3)
                hamming(zero_word(3), one_word(3)) = 3
                hamming(x, y) = hamming(one_word(3), zero_word(3))
                hamming(x, y) = 3
                3 <= hamming(x, y)
            }
            if not (y = zero_word(3)) {
                y = one_word(3)
                x = y
                false
            }
            3 <= hamming(x, y)
        }
        3 <= hamming(x, y)
    }
    repetition_code_three.contains(x) and repetition_code_three.contains(y) and x != y implies 3 <= hamming(x, y)
}

/// The minimum distance of the repetition code `{000, 111}` is three.
theorem repetition_three_min_distance {
    min_distance_at_least(repetition_code_three, 3)
} by {
    forall(x: List[Bool], y: List[Bool]) { repetition_three_pair_distance(x, y) }
    min_distance_at_least(repetition_code_three, 3) = forall(x: List[Bool], y: List[Bool]) {
        repetition_code_three.contains(x) and repetition_code_three.contains(y) and x != y implies 3 <= hamming(x, y)
    }
    min_distance_at_least(repetition_code_three, 3)
}

// The general sphere-packing (Hamming) bound.  A code of length `n` with
// minimum distance `d` and size `M` satisfies
//
//     M * sum_{i = 0}^{floor((d - 1) / 2)} C(n, i) <= 2^n .
//
// The proof counts the words within Hamming distance `t = floor((d - 1) / 2)`
// of each codeword: every ball of radius `t` has `sum_{i = 0}^{t} C(n, i)`
// words (choose the differing positions), the balls are disjoint because the
// minimum distance is `d > 2t`, and all balls lie among the `2^n` words of
// length `n`.  Formalizing the general statement requires a counting argument
// for balls and disjointness of the ball family; the bound is proved below
// for the small case `n = 3`, `d = 3`, where `t = 1` and
// `sum_{i = 0}^{1} C(3, i) = 1 + 3 = 4`, so the bound reads `M * 4 <= 8`.
//
// Likewise, for a binary `[n, k]` code (a code with `2^k` words) the bound
// takes the form
//
//     2^k * sum_{i = 0}^{floor((d - 1) / 2)} C(n, i) <= 2^n ,
//
// which for `n = 3`, `d = 3` reads `2^k * (1 + 3) <= 8`, hence `k <= 1`.

/// The number of words within Hamming distance one of a word of length three
/// is `1 + 3 = 4`.
theorem hamming_ball_radius_one_size_three {
    3.binom(0) + 3.binom(1) = 4
} by {
    choose_zero(3)
    choose_one(3)
    3.binom(0) = 1
    3.binom(1) = 3
}

/// Two to the third power is eight.
theorem two_pow_three {
    2.pow(3) = 8
} by {
    2.pow(3) = 2 * 2.pow(2)
    2.pow(2) = 2 * 2.pow(1)
    2.pow(1) = 2 * 2.pow(0)
    2.pow(0) = 1
}

/// A natural number `m` with `m * 4 <= 8` satisfies `m <= 2`.
theorem m_times_four_le_eight_imp_le_two(m: Nat) {
    m * 4 <= 8 implies m <= 2
} by {
    if m * 4 <= 8 {
        trichotomy(m, 2)
        if m < 2 or 2 < m or m = 2 {
            if m < 2 {
                lt_imp_lte_suc(m, 2)
                m.suc <= 2
                lte_cancel_suc(m, 1)
                m <= 1
                lte_trans(m, 1, 2)
                m <= 2
            }
            if not (m < 2) {
                if 2 < m or m = 2 {
                    if 2 < m {
                        lte_mul_both(4, 2, m)
                        4 * 2 <= 4 * m
                        8 <= 4 * m
                        m * 4 = 4 * m
                        4 * m <= 8
                        lte_antisymm(8, 4 * m)
                        8 = 4 * m
                        4 != 0
                        mul_cancel_left(4, m, 2)
                        4 * m = 4 * 2
                        m = 2
                        2 < 2
                        false
                    }
                    if not (2 < m) {
                        m = 2
                        m <= 2
                    }
                    m <= 2
                }
                m <= 2
            }
        }
        m <= 2
    }
}

/// The sphere-packing bound for length three and minimum distance three: a
/// code of size `m` with minimum distance three satisfies `m * (1 + 3) <= 8`,
/// hence `m <= 2`.
theorem sphere_packing_bound_three(m: Nat) {
    m * (3.binom(0) + 3.binom(1)) <= 2.pow(3) implies m <= 2
} by {
    if m * (3.binom(0) + 3.binom(1)) <= 2.pow(3) {
        choose_zero(3)
        choose_one(3)
        3.binom(0) = 1
        3.binom(1) = 3
        m * (3.binom(0) + 3.binom(1)) = m * 4
        two_pow_three
        2.pow(3) = 8
        m * 4 <= 8
        m_times_four_le_eight_imp_le_two(m)
        m <= 2
    }
}

/// The repetition code `{000, 111}` attains the sphere-packing bound for
/// length three: its size two satisfies `2 * (1 + 3) <= 8`.
theorem repetition_three_satisfies_bound {
    2 * (3.binom(0) + 3.binom(1)) <= 2.pow(3)
} by {
    choose_zero(3)
    choose_one(3)
    3.binom(0) = 1
    3.binom(1) = 3
    2 * (3.binom(0) + 3.binom(1)) = 2 * 4
    2 * 4 = 8
    lte_ref(8)
    8 <= 8
    2 * (3.binom(0) + 3.binom(1)) <= 8
    two_pow_three
    2.pow(3) = 8
    2 * (3.binom(0) + 3.binom(1)) <= 2.pow(3)
}

/// A single bit flip of `000` stays within distance one of `000`.
theorem flip_first_distance_one {
    hamming(flip_first_three, zero_word(3)) = Nat.1
} by {
    flip_first_three = List.cons(true, zero_word(2))
    zero_word(3) = List.cons(false, zero_word(2))
    hamming_cons_eq(zero_word(2), zero_word(2), true, false)
    hamming(List.cons(true, zero_word(2)), List.cons(false, zero_word(2))) = bit_diff(true, false) + hamming(zero_word(2), zero_word(2))
    hamming(flip_first_three, zero_word(3)) = bit_diff(true, false) + hamming(zero_word(2), zero_word(2))
    bit_diff(true, false) = Nat.1
    hamming_self(zero_word(2))
    hamming(zero_word(2), zero_word(2)) = Nat.0
    hamming(flip_first_three, zero_word(3)) = Nat.1
}

/// A single bit flip of `000` stays within distance one of `000`.
theorem flip_second_distance_one {
    hamming(flip_second_three, zero_word(3)) = Nat.1
} by {
    flip_second_three = List.cons(false, List.cons(true, zero_word(1)))
    zero_word(3) = List.cons(false, List.cons(false, zero_word(1)))
    hamming_cons_eq(List.cons(true, zero_word(1)), List.cons(false, zero_word(1)), false, false)
    hamming(List.cons(false, List.cons(true, zero_word(1))), List.cons(false, List.cons(false, zero_word(1)))) = bit_diff(false, false) + hamming(List.cons(true, zero_word(1)), List.cons(false, zero_word(1)))
    hamming_cons_eq(zero_word(1), zero_word(1), true, false)
    hamming(List.cons(true, zero_word(1)), List.cons(false, zero_word(1))) = bit_diff(true, false) + hamming(zero_word(1), zero_word(1))
    bit_diff(false, false) = Nat.0
    bit_diff(true, false) = Nat.1
    hamming_self(zero_word(1))
    hamming(zero_word(1), zero_word(1)) = Nat.0
    hamming(flip_second_three, zero_word(3)) = Nat.1
}

/// A single bit flip of `000` stays within distance one of `000`.
theorem flip_third_distance_one {
    hamming(flip_third_three, zero_word(3)) = Nat.1
} by {
    flip_third_three = List.cons(false, List.cons(false, List.cons(true, zero_word(0))))
    zero_word(3) = List.cons(false, zero_word(2))
    zero_word(2) = List.cons(false, zero_word(1))
    zero_word(1) = List.cons(false, zero_word(0))
    zero_word(3) = List.cons(false, List.cons(false, List.cons(false, zero_word(0))))
    hamming_cons_eq(List.cons(false, List.cons(true, zero_word(0))), List.cons(false, List.cons(false, zero_word(0))), false, false)
    hamming(List.cons(false, List.cons(false, List.cons(true, zero_word(0)))), List.cons(false, List.cons(false, List.cons(false, zero_word(0))))) = bit_diff(false, false) + hamming(List.cons(false, List.cons(true, zero_word(0))), List.cons(false, List.cons(false, zero_word(0))))
    hamming_cons_eq(List.cons(true, zero_word(0)), List.cons(false, zero_word(0)), false, false)
    hamming(List.cons(false, List.cons(true, zero_word(0))), List.cons(false, List.cons(false, zero_word(0)))) = bit_diff(false, false) + hamming(List.cons(true, zero_word(0)), List.cons(false, zero_word(0)))
    hamming_cons_eq(zero_word(0), zero_word(0), true, false)
    hamming(List.cons(true, zero_word(0)), List.cons(false, zero_word(0))) = bit_diff(true, false) + hamming(zero_word(0), zero_word(0))
    bit_diff(false, false) = Nat.0
    bit_diff(true, false) = Nat.1
    hamming_self(zero_word(0))
    hamming(zero_word(0), zero_word(0)) = Nat.0
    hamming(flip_third_three, zero_word(3)) = Nat.1
}

/// A single bit flip of `000` has length three.
theorem flip_first_length {
    flip_first_three.length = 3
} by {
    flip_first_three = List.cons(true, zero_word(2))
    List.cons(true, zero_word(2)).length = zero_word(2).length.suc
    zero_word_length(2)
    zero_word(2).length = 2
    flip_first_three.length = 3
}

/// A single bit flip of `000` has length three.
theorem flip_second_length {
    flip_second_three.length = 3
} by {
    flip_second_three = List.cons(false, List.cons(true, zero_word(1)))
    List.cons(false, List.cons(true, zero_word(1))).length = List.cons(true, zero_word(1)).length.suc
    List.cons(true, zero_word(1)).length = zero_word(1).length.suc
    zero_word_length(1)
    zero_word(1).length = 1
    flip_second_three.length = 3
}

/// A single bit flip of `000` has length three.
theorem flip_third_length {
    flip_third_three.length = 3
} by {
    flip_third_three = List.cons(false, List.cons(false, List.cons(true, zero_word(0))))
    List.cons(false, List.cons(false, List.cons(true, zero_word(0)))).length = List.cons(false, List.cons(true, zero_word(0))).length.suc
    List.cons(false, List.cons(true, zero_word(0))).length = List.cons(true, zero_word(0)).length.suc
    List.cons(true, zero_word(0)).length = zero_word(0).length.suc
    zero_word_length(0)
    zero_word(0).length = 0
    flip_third_three.length = 3
}

/// Any word at Hamming distance one from `000` is closer to `000` than to
/// `111`.
theorem single_flip_closer(w: List[Bool]) {
    w.length = 3 and hamming(w, zero_word(3)) = Nat.1 implies
        hamming(w, zero_word(3)) < hamming(w, one_word(3))
} by {
    if w.length = 3 and hamming(w, zero_word(3)) = Nat.1 {
        zero_word_length(3)
        one_word_length(3)
        zero_word(3).length = w.length
        w.length = one_word(3).length
        hamming_triangle(zero_word(3), w, one_word(3))
        hamming(zero_word(3), one_word(3)) <= hamming(zero_word(3), w) + hamming(w, one_word(3))
        hamming_sym(w, zero_word(3))
        hamming(w, zero_word(3)) = hamming(zero_word(3), w)
        hamming(zero_word(3), w) = Nat.1
        hamming_zero_one(3)
        hamming(zero_word(3), one_word(3)) = 3
        3 <= Nat.1 + hamming(w, one_word(3))
        Nat.1 + hamming(w, one_word(3)) = hamming(w, one_word(3)).suc
        2.suc <= hamming(w, one_word(3)).suc
        lte_cancel_suc(2, hamming(w, one_word(3)))
        2 <= hamming(w, one_word(3))
        1 < 2
        lt_and_lte(1, 2, hamming(w, one_word(3)))
        1 < hamming(w, one_word(3))
        hamming(w, zero_word(3)) = Nat.1
        hamming(w, zero_word(3)) < hamming(w, one_word(3))
    }
}

/// A single bit flip of `000` is closer to `000` than to `111`.
theorem flip_first_closer {
    hamming(flip_first_three, zero_word(3)) < hamming(flip_first_three, one_word(3))
} by {
    flip_first_distance_one
    flip_first_length
    single_flip_closer(flip_first_three)
    hamming(flip_first_three, zero_word(3)) < hamming(flip_first_three, one_word(3))
}

/// A single bit flip of `000` is closer to `000` than to `111`.
theorem flip_second_closer {
    hamming(flip_second_three, zero_word(3)) < hamming(flip_second_three, one_word(3))
} by {
    flip_second_distance_one
    flip_second_length
    single_flip_closer(flip_second_three)
    hamming(flip_second_three, zero_word(3)) < hamming(flip_second_three, one_word(3))
}

/// A single bit flip of `000` is closer to `000` than to `111`.
theorem flip_third_closer {
    hamming(flip_third_three, zero_word(3)) < hamming(flip_third_three, one_word(3))
} by {
    flip_third_distance_one
    flip_third_length
    single_flip_closer(flip_third_three)
    hamming(flip_third_three, zero_word(3)) < hamming(flip_third_three, one_word(3))
}

/// A natural number at most one is zero or one.
theorem lte_one_cases(n: Nat) {
    n <= Nat.1 implies n = Nat.0 or n = Nat.1
} by {
    define p(m: Nat) -> Bool {
        m <= Nat.1 implies m = Nat.0 or m = Nat.1
    }
    p(Nat.0)
    forall(k: Nat) {
        if p(k) {
            if k.suc <= Nat.1 {
                lte_cancel_suc(k, 0)
                k <= Nat.0
                only_zero_lte_zero(k)
                k = Nat.0
                k.suc = Nat.1
                k.suc = Nat.0 or k.suc = Nat.1
            }
            p(k.suc)
        }
    }
    p(n)
}

/// The repetition code `{000, 111}` corrects one error: every word within
/// distance one of `000` is strictly closer to `000` than to `111`.
theorem repetition_three_corrects_one_error(w: List[Bool]) {
    w.length = 3 and hamming(w, zero_word(3)) <= Nat.1 implies
        hamming(w, zero_word(3)) < hamming(w, one_word(3))
} by {
    if w.length = 3 and hamming(w, zero_word(3)) <= Nat.1 {
        lte_one_cases(hamming(w, zero_word(3)))
        if hamming(w, zero_word(3)) = Nat.0 or hamming(w, zero_word(3)) = Nat.1 {
            if hamming(w, zero_word(3)) = Nat.0 {
                zero_word_length(3)
                hamming_zero_iff_eq(w, zero_word(3))
                w = zero_word(3)
                hamming_zero_one(3)
                hamming(w, one_word(3)) = 3
                0 < 3
                hamming(w, zero_word(3)) < hamming(w, one_word(3))
            }
            if not (hamming(w, zero_word(3)) = Nat.0) {
                hamming(w, zero_word(3)) = Nat.1
                single_flip_closer(w)
                hamming(w, zero_word(3)) < hamming(w, one_word(3))
            }
            hamming(w, zero_word(3)) < hamming(w, one_word(3))
        }
        hamming(w, zero_word(3)) < hamming(w, one_word(3))
    }
}

/// A power of two is at least one.
theorem two_pow_nonneg(n: Nat) {
    Nat.1 <= 2.pow(n)
} by {
    define p(m: Nat) -> Bool {
        Nat.1 <= 2.pow(m)
    }
    2.pow(Nat.0) = 1
    p(Nat.0)
    forall(k: Nat) {
        if p(k) {
            2.pow(k.suc) = 2 * 2.pow(k)
            lte_mul_both(2, 1, 2.pow(k))
            2 * 1 <= 2 * 2.pow(k)
            2 <= 2 * 2.pow(k)
            lte_trans(1, 2, 2 * 2.pow(k))
            1 <= 2 * 2.pow(k)
            p(k.suc)
        }
    }
    p(n)
}

/// A power of two with exponent at least two is at least four.
theorem two_pow_ge_four_of_ge_two(k: Nat) {
    2 <= k implies 4 <= 2.pow(k)
} by {
    if 2 <= k {
        lt_or_lte(2, k)
        if 2 < k or k <= 2 {
            if 2 < k {
                lt_diff(2, k)
                let c: Nat satisfy {
                    2 + c = k and c != 0
                }
                zero_or_suc(c)
                if c = 0 or exists(b: Nat) { c = b.suc } {
                    if c = 0 {
                        false
                    }
                    if not (c = 0) {
                        let d: Nat satisfy { c = d.suc }
                        k = 2 + d.suc
                        pow_add(2, 2, d.suc)
                        2.pow(2) * 2.pow(d.suc) = 2.pow(2 + d.suc)
                        2.pow(2) = 4
                        4 * 2.pow(d.suc) = 2.pow(2 + d.suc)
                        2.pow(2 + d.suc) = 2.pow(k)
                        two_pow_nonneg(d.suc)
                        1 <= 2.pow(d.suc)
                        lte_mul_both(4, 1, 2.pow(d.suc))
                        4 * 1 <= 4 * 2.pow(d.suc)
                        4 <= 4 * 2.pow(d.suc)
                        lte_trans(4, 4 * 2.pow(d.suc), 2.pow(k))
                        4 <= 2.pow(k)
                    }
                    4 <= 2.pow(k)
                }
                4 <= 2.pow(k)
            }
            if not (2 < k) {
                k <= 2
                lte_antisymm(2, k)
                2 = k
                k = 2
                2.pow(k) = 2.pow(2)
                2.pow(2) = 4
                2.pow(k) = 4
                lte_ref(4)
                4 <= 4
                4 <= 2.pow(k)
            }
            4 <= 2.pow(k)
        }
        4 <= 2.pow(k)
    }
}

/// A power of two at most two has exponent at most one.
theorem two_pow_le_two_imp_le_one(k: Nat) {
    2.pow(k) <= 2 implies k <= 1
} by {
    if 2.pow(k) <= 2 {
        lt_or_lte(1, k)
        if 1 < k or k <= 1 {
            if 1 < k {
                lt_imp_lte_suc(1, k)
                2 <= k
                two_pow_ge_four_of_ge_two(k)
                4 <= 2.pow(k)
                lte_trans(4, 2.pow(k), 2)
                4 <= 2
                lte_imp_not_lt(4, 2)
                not (2 < 4)
                lt_suc(2)
                2 < 3
                lt_imp_lt_suc(2, 3)
                2 < 4
                false
            }
            if not (1 < k) {
                k <= 1
            }
            k <= 1
        }
        k <= 1
    }
}

/// The sphere-packing bound for an `[n, k]` code of length three and minimum
/// distance three: `2^k * (1 + 3) <= 8`, hence `k <= 1`.
theorem sphere_packing_bound_k_three(k: Nat) {
    2.pow(k) * (3.binom(0) + 3.binom(1)) <= 2.pow(3) implies k <= 1
} by {
    if 2.pow(k) * (3.binom(0) + 3.binom(1)) <= 2.pow(3) {
        sphere_packing_bound_three(2.pow(k))
        2.pow(k) <= 2
        two_pow_le_two_imp_le_one(k)
        k <= 1
    }
}
