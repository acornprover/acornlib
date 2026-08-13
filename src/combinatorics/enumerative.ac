// Enumerative combinatorics: the classical identities of the binomial
// coefficients.
//
// This file collects the standard identities of enumerative combinatorics in
// their classical shapes:
//
//   (a) Pascal's identity            C(n, k) = C(n - 1, k - 1) + C(n - 1, k)
//   (b) the hockey-stick identity    sum_{i=k}^{n} C(i, k) = C(n + 1, k + 1),
//       with the k = 1 case          sum_{i=1}^{n} i = C(n + 1, 2)
//       (the triangular numbers)
//   (c) Vandermonde's identity       sum_j C(m, j) C(n, r - j) = C(m + n, r),
//       with the r = 1 case          C(m, 0) C(n, 1) + C(m, 1) C(n, 0) = m + n
//   (d) the binomial theorem         (1 + x)^n = sum_k C(n, k) x^k,
//       with the n = 2 case          (1 + x)^2 = 1 + 2x + x^2
//   (e) the subset-counting identity sum_k C(n, k) = 2^n
//
// The general forms are already proved in `binomial.ac` (for the naturals)
// and in `comm_ring/binomial.ac` (for commutative rings, hence for the
// reals); this file restates them in their classical shapes and proves the
// special cases.

from nat import Nat, mul_cancel_left, from_nat, from_nat_zero, from_nat_one,
    from_nat_add, pow_zero, pow_one, one_pow, add_sub, sub_zero, sub_self,
    lt_imp_lte_suc, lte_trans
from list import partial, partial_zero, partial_one, partial_split_last,
    partial_pointwise_eq
from data.basic.functions import compose
from combinatorics import binom, pascal_suc_unbounded, choose_zero, choose_one,
    choose_n, choose_out_of_bounds, hockey_stick, hockey_stick_term,
    vandermonde, vandermonde_term, binom_row_sum, triangular, triangular_zero,
    triangular_doubled, binom_two_double
from comm_ring import binomial_term, binomial
from real import Real, two

numerals Nat
numerals Real

// ---------------------------------------------------------------------------
// (a) Pascal's identity
// ---------------------------------------------------------------------------

/// Pascal's identity: each entry of Pascal's triangle is the sum of the two
/// entries directly above it:
/// `C(n, k) = C(n - 1, k - 1) + C(n - 1, k)` for `0 < k <= n`.
///
/// Restates the successor form `pascal_suc_unbounded` of `binomial.ac`.
theorem pascal_sub_form(n: Nat, k: Nat) {
    Nat.0 < k and k <= n implies
        n.binom(k) = (n - Nat.1).binom(k - Nat.1) + (n - Nat.1).binom(k)
} by {
    if Nat.0 < k and k <= n {
        lt_imp_lte_suc(Nat.0, k)
        Nat.1 <= k
        lte_trans(Nat.1, k, n)
        Nat.1 <= n
        add_sub(n, Nat.1)
        n - Nat.1 + Nat.1 = n
        add_sub(k, Nat.1)
        k - Nat.1 + Nat.1 = k
        pascal_suc_unbounded(n - Nat.1, k - Nat.1)
        (n - Nat.1).suc.binom((k - Nat.1).suc) =
            (n - Nat.1).binom(k - Nat.1) + (n - Nat.1).binom((k - Nat.1).suc)
        (n - Nat.1) + Nat.1 = (n - Nat.1).suc
        (k - Nat.1) + Nat.1 = (k - Nat.1).suc
        (n - Nat.1).suc = n
        (k - Nat.1).suc = k
        (n - Nat.1).binom((k - Nat.1).suc) = (n - Nat.1).binom(k)
        n.binom(k) = (n - Nat.1).binom(k - Nat.1) + (n - Nat.1).binom(k)
    }
}

// ---------------------------------------------------------------------------
// (b) The hockey-stick identity
// ---------------------------------------------------------------------------

/// The hockey-stick identity: the sum of the binomial coefficients down a
/// diagonal of Pascal's triangle is the coefficient immediately below it:
/// `sum_{i=k}^{n} C(i, k) = C(n + 1, k + 1)`.
///
/// Restates `hockey_stick` of `binomial.ac`, whose `j`-th summand
/// `hockey_stick_term(k, j) = C(k + j, k)` covers `i = k + j` from `k` to `n`.
theorem hockey_stick_sum_form(k: Nat, n: Nat) {
    k <= n implies
        partial(hockey_stick_term(k), (n - k).suc) = (n + Nat.1).binom(k + Nat.1)
} by {
    if k <= n {
        hockey_stick(k, n - k)
        partial(hockey_stick_term(k), (n - k).suc) = (k + (n - k).suc).binom(k.suc)
        (n - k).suc = (n - k) + Nat.1
        k + ((n - k) + Nat.1) = (k + (n - k)) + Nat.1
        (n - k) + k = n
        k + (n - k) = (n - k) + k
        (k + (n - k)) + Nat.1 = n + Nat.1
        k + (n - k).suc = n + Nat.1
        k.suc = k + Nat.1
        partial(hockey_stick_term(k), (n - k).suc) = (n + Nat.1).binom(k + Nat.1)
    }
}

/// The `i`-th summand of `sum_{i=1}^{n} i`, indexed from zero: the successor
/// of `i`, so `partial(successor_summand, n) = 1 + 2 + ... + n`.
define successor_summand(i: Nat) -> Nat {
    i.suc
}

/// The k = 1 case of the hockey-stick identity: the sum of the first `n`
/// positive integers — the `n`-th triangular number — equals `C(n + 1, 2)`:
/// `sum_{i=1}^{n} i = C(n + 1, 2)`.
///
/// The `j`-th summand `C(1 + j, 1) = 1 + j` is `successor_summand(j)`, so the
/// sum is `partial(successor_summand, n) = 1 + 2 + ... + n`.
theorem hockey_stick_k1(n: Nat) {
    partial(successor_summand, n) = (n + Nat.1).binom(Nat.2)
} by {
    if n = Nat.0 {
        partial_zero[Nat](successor_summand)
        partial(successor_summand, Nat.0) = Nat.0
        Nat.0 + Nat.1 = Nat.1
        choose_out_of_bounds(Nat.1, Nat.2)
        Nat.1 < Nat.2
        Nat.1.binom(Nat.2) = Nat.0
        partial(successor_summand, n) = (n + Nat.1).binom(Nat.2)
    } else {
        let m: Nat satisfy { m.suc = n }
        m = n - Nat.1
        hockey_stick(Nat.1, m)
        partial(hockey_stick_term(Nat.1), m.suc) = (Nat.1 + m.suc).binom(Nat.2)
        Nat.1 + m.suc = n + Nat.1
        partial(hockey_stick_term(Nat.1), n) = (n + Nat.1).binom(Nat.2)
        forall(j: Nat) {
            if j < n {
                hockey_stick_term(Nat.1, j) = (Nat.1 + j).binom(Nat.1)
                choose_one(Nat.1 + j)
                (Nat.1 + j).binom(Nat.1) = Nat.1 + j
                j.suc = Nat.1 + j
                successor_summand(j) = j.suc
                hockey_stick_term(Nat.1, j) = successor_summand(j)
            }
        }
        partial_pointwise_eq[Nat](hockey_stick_term(Nat.1), successor_summand, n)
        partial(hockey_stick_term(Nat.1), n) = partial(successor_summand, n)
        partial(successor_summand, n) = (n + Nat.1).binom(Nat.2)
    }
}

/// The `n`-th triangular number `0 + 1 + ... + n` equals `C(n + 1, 2)`.
///
/// Proved from the closed form `2 * triangular(n) = n * (n + 1)` of
/// `binomial.ac` and the double of `C(n + 1, 2)`, cancelling the factor two.
theorem triangular_binom(n: Nat) {
    triangular(n) = (n + Nat.1).binom(Nat.2)
} by {
    if n = Nat.0 {
        triangular_zero
        triangular(Nat.0) = Nat.0
        Nat.0 + Nat.1 = Nat.1
        choose_out_of_bounds(Nat.1, Nat.2)
        Nat.1 < Nat.2
        Nat.1.binom(Nat.2) = Nat.0
        triangular(n) = (n + Nat.1).binom(Nat.2)
    } else {
        Nat.0 < n
        Nat.1 <= n
        binom_two_double(n + Nat.1)
        Nat.2 <= n + Nat.1
        Nat.2 * (n + Nat.1).binom(Nat.2) = (n + Nat.1) * ((n + Nat.1) - Nat.1)
        (n + Nat.1) - Nat.1 = n
        Nat.2 * (n + Nat.1).binom(Nat.2) = (n + Nat.1) * n
        triangular_doubled(n)
        Nat.2 * triangular(n) = n * (n + Nat.1)
        n * (n + Nat.1) = (n + Nat.1) * n
        Nat.2 * triangular(n) = Nat.2 * (n + Nat.1).binom(Nat.2)
        mul_cancel_left(Nat.2, triangular(n), (n + Nat.1).binom(Nat.2))
        triangular(n) = (n + Nat.1).binom(Nat.2)
    }
}

// ---------------------------------------------------------------------------
// (c) Vandermonde's identity
// ---------------------------------------------------------------------------

/// Vandermonde's identity: the convolution of the binomial rows `m` and `n`
/// is the binomial row `m + n`:
/// `sum_{j=0}^{r} C(m, j) C(n, r - j) = C(m + n, r)`.
///
/// Restates `vandermonde` of `binomial.ac`; the sum runs over
/// `j = 0, ..., r` since `C(n, r - j)` vanishes for `r - j > n`.
theorem vandermonde_sum_form(m: Nat, n: Nat, r: Nat) {
    partial(vandermonde_term(m, n, r), r.suc) = (m + n).binom(r)
} by {
    vandermonde(m, n, r)
    partial(vandermonde_term(m, n, r), r.suc) = (m + n).binom(r)
}

/// The r = 1 case of Vandermonde's identity:
/// `C(m, 0) C(n, 1) + C(m, 1) C(n, 0) = C(m + n, 1)`.
theorem vandermonde_r1(m: Nat, n: Nat) {
    partial(vandermonde_term(m, n, Nat.1), Nat.2) = (m + n).binom(Nat.1)
} by {
    vandermonde(m, n, Nat.1)
    partial(vandermonde_term(m, n, Nat.1), Nat.2) = (m + n).binom(Nat.1)
}

/// The r = 1 case of Vandermonde's identity evaluates to `m + n`:
/// `C(m, 0) C(n, 1) + C(m, 1) C(n, 0) = m + n`.
theorem vandermonde_r1_eval(m: Nat, n: Nat) {
    partial(vandermonde_term(m, n, Nat.1), Nat.2) = m + n
} by {
    vandermonde_r1(m, n)
    partial(vandermonde_term(m, n, Nat.1), Nat.2) = (m + n).binom(Nat.1)
    choose_one(m + n)
    (m + n).binom(Nat.1) = m + n
    partial(vandermonde_term(m, n, Nat.1), Nat.2) = m + n
}

// ---------------------------------------------------------------------------
// (d) The binomial theorem for (1 + x)^n
// ---------------------------------------------------------------------------

/// The binomial theorem for `(1 + x)^n` over the reals:
/// `(1 + x)^n = sum_{k=0}^{n} C(n, k) x^k`.
///
/// The `k`-th summand `binomial_term[Real](x, Real.1, n, k) =
/// C(n, k) x^k 1^(n-k)` equals `C(n, k) x^k`.  Instantiation of the binomial
/// theorem for commutative rings, `binomial`, at the reals.
theorem binomial_one_plus_x(x: Real, n: Nat) {
    (Real.1 + x).pow(n) = partial(binomial_term[Real](x, Real.1, n), n.suc)
} by {
    binomial[Real](x, Real.1, n)
    (x + Real.1).pow(n) = partial(binomial_term[Real](x, Real.1, n), n.suc)
    (x + Real.1).pow(n) = (Real.1 + x).pow(n)
    (Real.1 + x).pow(n) = partial(binomial_term[Real](x, Real.1, n), n.suc)
}

/// The n = 2 case of the binomial theorem:
/// `(1 + x)^2 = 1 + 2x + x^2`.
theorem binomial_square(x: Real) {
    (Real.1 + x).pow(Nat.2) = Real.1 + two * x + x.pow(Nat.2)
} by {
    binomial_one_plus_x(x, Nat.2)
    (Real.1 + x).pow(Nat.2) = partial(binomial_term[Real](x, Real.1, Nat.2), Nat.3)
    partial_split_last[Real](binomial_term[Real](x, Real.1, Nat.2), Nat.2)
    partial(binomial_term[Real](x, Real.1, Nat.2), Nat.3) =
        partial(binomial_term[Real](x, Real.1, Nat.2), Nat.2) +
        binomial_term[Real](x, Real.1, Nat.2, Nat.2)
    partial_split_last[Real](binomial_term[Real](x, Real.1, Nat.2), Nat.1)
    partial(binomial_term[Real](x, Real.1, Nat.2), Nat.2) =
        partial(binomial_term[Real](x, Real.1, Nat.2), Nat.1) +
        binomial_term[Real](x, Real.1, Nat.2, Nat.1)
    partial_one[Real](binomial_term[Real](x, Real.1, Nat.2))
    partial(binomial_term[Real](x, Real.1, Nat.2), Nat.1) =
        binomial_term[Real](x, Real.1, Nat.2, Nat.0)
    binomial_term[Real](x, Real.1, Nat.2, Nat.0) =
        from_nat[Real](Nat.2.binom(Nat.0)) * x.pow(Nat.0) * Real.1.pow(Nat.2 - Nat.0)
    binomial_term[Real](x, Real.1, Nat.2, Nat.1) =
        from_nat[Real](Nat.2.binom(Nat.1)) * x.pow(Nat.1) * Real.1.pow(Nat.2 - Nat.1)
    binomial_term[Real](x, Real.1, Nat.2, Nat.2) =
        from_nat[Real](Nat.2.binom(Nat.2)) * x.pow(Nat.2) * Real.1.pow(Nat.2 - Nat.2)
    // Simplify the constant coefficients.
    choose_zero(Nat.2)
    Nat.2.binom(Nat.0) = Nat.1
    choose_one(Nat.2)
    Nat.2.binom(Nat.1) = Nat.2
    choose_n(Nat.2)
    Nat.2.binom(Nat.2) = Nat.1
    from_nat_one[Real]
    from_nat[Real](Nat.1) = Real.1
    from_nat_add[Real](Nat.1, Nat.1)
    from_nat[Real](Nat.1 + Nat.1) = from_nat[Real](Nat.1) + from_nat[Real](Nat.1)
    Nat.1 + Nat.1 = Nat.2
    from_nat[Real](Nat.2) = Real.1 + Real.1
    Real.1 + Real.1 = two
    from_nat[Real](Nat.2) = two
    // Simplify the powers.
    pow_zero[Real](x)
    x.pow(Nat.0) = Real.1
    pow_one[Real](x)
    x.pow(Nat.1) = x
    sub_zero(Nat.2)
    Nat.2 - Nat.0 = Nat.2
    one_pow[Real](Nat.2)
    Real.1.pow(Nat.2) = Real.1
    Real.1.pow(Nat.2 - Nat.0) = Real.1
    Nat.2 - Nat.1 = Nat.1
    one_pow[Real](Nat.1)
    Real.1.pow(Nat.1) = Real.1
    Real.1.pow(Nat.2 - Nat.1) = Real.1
    Nat.2 - Nat.2 = Nat.0
    one_pow[Real](Nat.0)
    Real.1.pow(Nat.0) = Real.1
    Real.1.pow(Nat.2 - Nat.2) = Real.1
    // The three terms evaluate to 1, 2x and x^2.
    binomial_term[Real](x, Real.1, Nat.2, Nat.0) = Real.1
    binomial_term[Real](x, Real.1, Nat.2, Nat.1) = two * x
    binomial_term[Real](x, Real.1, Nat.2, Nat.2) = x.pow(Nat.2)
    partial(binomial_term[Real](x, Real.1, Nat.2), Nat.3) =
        Real.1 + two * x + x.pow(Nat.2)
    (Real.1 + x).pow(Nat.2) = Real.1 + two * x + x.pow(Nat.2)
}

// ---------------------------------------------------------------------------
// (e) The number of subsets of an n-element set
// ---------------------------------------------------------------------------

/// The number of subsets of an `n`-element set is `2^n`: the sum of the
/// binomial coefficients in row `n` equals `2^n`,
/// `sum_{k=0}^{n} C(n, k) = 2^n`.
///
/// Restates `binom_row_sum` of `binomial.ac`.  The set-theoretic form — a
/// finite set of cardinality `n` has a powerset of cardinality `2^n` — is
/// `finite_powerset_cardinality` in `finite_set/powerset.ac`.
theorem subset_count_binom_sum(n: Nat) {
    partial(n.binom, n.suc) = Nat.2.pow(n)
} by {
    binom_row_sum(n)
    partial(n.binom, n.suc) = Nat.2.pow(n)
}

// ---------------------------------------------------------------------------
// The Catalan numbers
// ---------------------------------------------------------------------------

// The n-th Catalan number counts the number of ways to triangulate a convex
// (n + 2)-gon, the number of Dyck words of length 2n, and the number of full
// binary trees with n + 1 leaves, and satisfies the recurrence C_0 = 1 and
// C_{n+1} = sum_{i=0}^{n} C_i C_{n-i}.
//
// The Catalan numbers, the closed form C_n * (n + 1) = binom(2n, n), and the
// convolution recurrence are proved in `combinatorics/catalan.ac`:
//
//     define catalan(n: Nat) -> Nat { ... }
//     theorem catalan_mul_suc(n: Nat) { ... }       // the closed form
//     theorem catalan_recurrence(n: Nat) { ... }    // the recurrence
