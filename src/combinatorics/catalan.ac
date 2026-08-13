/// The Catalan numbers.
///
/// The `n`-th Catalan number `C_n` counts the number of Dyck words of length
/// `2n`, the number of ways to triangulate a convex `(n + 2)`-gon, and the
/// number of full binary trees with `n + 1` leaves.  It is defined here by
/// the difference-of-binomials closed form
///
///     C_n = binom(2n, n) - binom(2n, n + 1),
///
/// which is the integer form of the classical closed form
/// `C_n = binom(2n, n) / (n + 1)`.
///
/// The file proves the closed form `C_n * (n + 1) = binom(2n, n)`, the
/// classical convolution recurrence
///
///     C_{n+1} = sum_{k=0}^{n} C_k * C_{n-k},
///
/// and the small values.  The recurrence proof follows the classical
/// generating-function-free route: the auxiliary family
///
///     T_n(j) = sum_{k=0}^{n} C_k * binom(2(n - k), n - k + j)
///            = binom(2n, n + j) + binom(2n, n + j + 1)
///
/// is proved for every `j` by induction on `n`, and the `j = 0` member gives
/// `sum_k C_k * binom(2(n - k), n - k) = binom(2n, n) + binom(2n, n + 1)`,
/// from which the recurrence follows by the closed form.
///
/// The file also records, in `Nat`, the two subtraction-distribution lemmas
/// that the subtraction-heavy proofs need: `(a - b) * c = a * c - b * c` and
/// `(a + c) - (b + c) = a - b`.

from nat import Nat, alt_induction, add_sub, sub_lt, sub_self, sub_zero, add_cancels_right,
    lte_trans, lte_suc_suc, lt_imp_lte_suc, add_imp_sub, add_comm, add_assoc,
    mul_comm, mul_assoc, add_cancels_left, lt_imp_lt_suc, lt_or_lte, lt_trans,
    add_imp_sub_left, lt_add_left, lte_add_right, lte_ref, lt_and_lte, distrib_right, distrib_left, lt_mul_both,
    mul_zero_right, mul_suc_right, mul_two_left, mul_cancel_left, add_comm_4, add_one_right, add_suc_right, pow_zero, pow_one
from list import partial, partial_split_last, partial_drop_first,
    partial_pointwise_eq, partial_add, partial_scalar_mul, partial_reverse,
    reverse_index
from data.basic.functions import compose
from algebra.semigroup import mul_fn
from algebra.add_semigroup import add_fn
from combinatorics import binom, choose_zero, choose_n, choose_one,
    choose_out_of_bounds, binom_complement_absorption, binom_suc_le_binom,
    pascal_suc_unbounded, choose_symm_sub_form

numerals Nat

// ---------------------------------------------------------------------------
// Natural-number subtraction lemmas
// ---------------------------------------------------------------------------

/// Successor subtraction: `(a + 1) - (b + 1) = a - b`.
theorem nat_suc_sub_suc(a: Nat, b: Nat) {
    a.suc - b.suc = a - b
} by {
    if b <= a {
        add_sub(a, b)
        a - b + b = a
        add_sub(a.suc, b.suc)
        b.suc <= a.suc
        a.suc - b.suc + b.suc = a.suc
        (a - b) + b.suc = (a - b) + b + Nat.1
        (a - b) + b + Nat.1 = a + Nat.1
        (a - b) + b.suc = a.suc
        a.suc - b.suc + b.suc = (a - b) + b.suc
        add_cancels_right(a.suc - b.suc, a - b, b.suc)
        a.suc - b.suc = a - b
    } else {
        not b <= a
        lt_or_lte(b, a)
        a < b
        sub_lt(a, b)
        a - b = Nat.0
        lt_imp_lt_suc(a, b)
        a.suc < b.suc
        sub_lt(a.suc, b.suc)
        a.suc - b.suc = Nat.0
        a.suc - b.suc = a - b
    }
}

/// Subtraction distributes over addition: `(a + c) - (b + c) = a - b`.
theorem nat_add_sub_cancel(a: Nat, b: Nat, c: Nat) {
    (a + c) - (b + c) = a - b
} by {
    if b <= a {
        add_sub(a, b)
        a - b + b = a
        add_sub(a + c, b + c)
        b + c <= a + c
        (a + c) - (b + c) + (b + c) = a + c
        (a - b) + (b + c) = (a - b) + b + c
        (a - b) + b + c = a + c
        (a - b) + (b + c) = a + c
        (a + c) - (b + c) + (b + c) = (a - b) + (b + c)
        add_cancels_right((a + c) - (b + c), a - b, b + c)
        (a + c) - (b + c) = a - b
    } else {
        not b <= a
        lt_or_lte(b, a)
        a < b
        sub_lt(a, b)
        a - b = Nat.0
        lt_add_left(c, a, b)
        a < b implies c + a < c + b
        c + a < c + b
        add_comm(c, a)
        c + a = a + c
        add_comm(c, b)
        c + b = b + c
        a + c < b + c
        sub_lt(a + c, b + c)
        (a + c) - (b + c) = Nat.0
        (a + c) - (b + c) = a - b
    }
}

/// Multiplication distributes over subtraction on the right:
/// `(a - b) * c = a * c - b * c`.
theorem nat_mul_sub_distrib_right(a: Nat, b: Nat, c: Nat) {
    (a - b) * c = a * c - b * c
} by {
    if b <= a {
        add_sub(a, b)
        a - b + b = a
        (a - b + b) * c = a * c
        distrib_right(a - b, b, c)
        (a - b + b) * c = (a - b) * c + b * c
        (a - b) * c + b * c = a * c
        add_imp_sub((a - b) * c, b * c, a * c)
        a * c - b * c = (a - b) * c
    } else {
        not b <= a
        lt_or_lte(b, a)
        a < b
        sub_lt(a, b)
        a - b = Nat.0
        (a - b) * c = Nat.0 * c
        Nat.0 * c = Nat.0
        if c = Nat.0 {
            a * c = a * Nat.0
            a * Nat.0 = Nat.0
            b * c = b * Nat.0
            b * Nat.0 = Nat.0
            a * c - b * c = Nat.0 - Nat.0
            sub_self(a * c)
            a * c - a * c = Nat.0
            a * c - b * c = Nat.0
        } else {
            lt_mul_both(c, a, b)
            c != Nat.0 and a < b
            c * a < c * b
            mul_comm(c, a)
            c * a = a * c
            mul_comm(c, b)
            c * b = b * c
            a * c < b * c
            sub_lt(a * c, b * c)
            a * c - b * c = Nat.0
        }
        (a - b) * c = a * c - b * c
    }
}

/// Multiplication distributes over subtraction on the left:
/// `a * (b - c) = a * b - a * c`.
theorem nat_mul_sub_distrib_left(a: Nat, b: Nat, c: Nat) {
    a * (b - c) = a * b - a * c
} by {
    mul_comm(a, b - c)
    a * (b - c) = (b - c) * a
    nat_mul_sub_distrib_right(b, c, a)
    (b - c) * a = b * a - c * a
    mul_comm(b, a)
    b * a = a * b
    mul_comm(c, a)
    c * a = a * c
    a * (b - c) = a * b - a * c
}

// ---------------------------------------------------------------------------
// The Catalan numbers
// ---------------------------------------------------------------------------

/// The `n`-th Catalan number, the difference of adjacent central binomial
/// coefficients `binom(2n, n) - binom(2n, n + 1)`.
define catalan(n: Nat) -> Nat {
    (n + n).binom(n) - (n + n).binom(n + Nat.1)
}

/// C_0 = 1.
theorem catalan_zero {
    catalan(Nat.0) = Nat.1
} by {
    catalan(Nat.0) = (Nat.0 + Nat.0).binom(Nat.0) - (Nat.0 + Nat.0).binom(Nat.0 + Nat.1)
    Nat.0 + Nat.0 = Nat.0
    Nat.0 + Nat.1 = Nat.1
    catalan(Nat.0) = Nat.0.binom(Nat.0) - Nat.0.binom(Nat.1)
    choose_n(Nat.0)
    Nat.0.binom(Nat.0) = Nat.1
    choose_out_of_bounds(Nat.0, Nat.1)
    Nat.0 < Nat.1
    Nat.0.binom(Nat.1) = Nat.0
    Nat.1 - Nat.0 = Nat.1
    catalan(Nat.0) = Nat.1
}

/// C_1 = 1.
theorem catalan_one {
    catalan(Nat.1) = Nat.1
} by {
    catalan(Nat.1) = (Nat.1 + Nat.1).binom(Nat.1) - (Nat.1 + Nat.1).binom(Nat.1 + Nat.1)
    Nat.1 + Nat.1 = Nat.2
    catalan(Nat.1) = Nat.2.binom(Nat.1) - Nat.2.binom(Nat.2)
    choose_one(Nat.2)
    Nat.2.binom(Nat.1) = Nat.2
    choose_n(Nat.2)
    Nat.2.binom(Nat.2) = Nat.1
    Nat.2 - Nat.1 = Nat.1
    catalan(Nat.1) = Nat.1
}

// ---------------------------------------------------------------------------
// The central binomial coefficients
// ---------------------------------------------------------------------------

/// For n > 0, n < n + n.
theorem lt_n_n_plus_n(n: Nat) {
    Nat.0 < n implies n < n + n
} by {
    if Nat.0 < n {
        lt_imp_lte_suc(Nat.0, n)
        Nat.1 <= n
        lte_trans(Nat.1, n, n + n)
        n <= n + n
        n < n + n
    }
}

/// The successor of n is at most n + n + 1.
theorem nat_suc_lte_double_suc(n: Nat) {
    n.suc <= n + n + Nat.1
} by {
    n.suc = n + Nat.1
    n.suc + n = (n + Nat.1) + n
    (n + Nat.1) + n = n + n + Nat.1
    n.suc + n = n + n + Nat.1
    exists(c: Nat) {
        n.suc + c = n + n + Nat.1
    }
}

/// (n - 1) + 1 = n when n is positive.
theorem nat_pred_add_one(n: Nat) {
    Nat.0 < n implies n - Nat.1 + Nat.1 = n
} by {
    if Nat.0 < n {
        lt_imp_lte_suc(Nat.0, n)
        Nat.1 <= n
        add_sub(n, Nat.1)
        n - Nat.1 + Nat.1 = n
    }
}

/// (n - 1).suc = n when n is positive.
theorem nat_pred_suc(n: Nat) {
    Nat.0 < n implies (n - Nat.1).suc = n
} by {
    if Nat.0 < n {
        nat_pred_add_one(n)
        n - Nat.1 + Nat.1 = n
        (n - Nat.1).suc = n - Nat.1 + Nat.1
        (n - Nat.1).suc = n
    }
}

/// For n > 0, n - 1 is at most n + n, witnessed by n + 1.
theorem nat_pred_lte_double(n: Nat) {
    Nat.0 < n implies n - Nat.1 <= n + n
} by {
    if Nat.0 < n {
        nat_pred_add_one(n)
        n - Nat.1 + Nat.1 = n
        (n - Nat.1) + n.suc = (n - Nat.1) + (Nat.1 + n)
        (n - Nat.1) + (Nat.1 + n) = (n - Nat.1 + Nat.1) + n
        (n - Nat.1 + Nat.1) + n = n + n
        (n - Nat.1) + n.suc = n + n
        exists(c: Nat) {
            (n - Nat.1) + c = n + n
        }
    }
}

/// For n > 0, n + n - (n - 1) = n + 1.
theorem nat_double_sub_pred(n: Nat) {
    Nat.0 < n implies n + n - (n - Nat.1) = n.suc
} by {
    if Nat.0 < n {
        nat_pred_add_one(n)
        n - Nat.1 + Nat.1 = n
        (n - Nat.1) + n.suc = (n - Nat.1) + (Nat.1 + n)
        (n - Nat.1) + (Nat.1 + n) = (n - Nat.1 + Nat.1) + n
        (n - Nat.1 + Nat.1) + n = n + n
        (n - Nat.1) + n.suc = n + n
        add_imp_sub_left(n - Nat.1, n.suc, n + n)
        n + n - (n - Nat.1) = n.suc
    }
}

/// The adjacent central binomial coefficients: for n > 0,
/// `(n + 1) * binom(2n, n + 1) = n * binom(2n, n)`.
theorem central_binom_adjacent(n: Nat) {
    Nat.0 < n implies
        n.suc * (n + n).binom(n.suc) = n * (n + n).binom(n)
} by {
    if Nat.0 < n {
        lt_n_n_plus_n(n)
        n < n + n
        binom_complement_absorption(n + n, n)
        ((n + n) - n) * (n + n).binom(n) = n.suc * (n + n).binom(n.suc)
        (n + n) - n = n
        n * (n + n).binom(n) = n.suc * (n + n).binom(n.suc)
    }
}

/// The adjacent central binomial coefficient is at most the central one.
theorem central_binom_adjacent_lte(n: Nat) {
    (n + n).binom(n.suc) <= (n + n).binom(n)
} by {
    binom_suc_le_binom(n + n, n)
    n + n <= Nat.2 * n
    Nat.2 * n = n + n
    (n + n).binom(n.suc) <= (n + n).binom(n)
}

/// The successor of a central binomial coefficient doubles the sum of the
/// central and adjacent coefficients:
/// `binom(2n + 2, n + 1) = 2 * (binom(2n, n) + binom(2n, n + 1))`.
theorem central_binom_suc(n: Nat) {
    (n.suc + n.suc).binom(n.suc) =
        Nat.2 * ((n + n).binom(n) + (n + n).binom(n.suc))
} by {
    if n = Nat.0 {
        (Nat.0.suc + Nat.0.suc).binom(Nat.0.suc) = Nat.2.binom(Nat.1)
        choose_one(Nat.2)
        Nat.2.binom(Nat.1) = Nat.2
        (Nat.0 + Nat.0).binom(Nat.0) = Nat.0.binom(Nat.0)
        choose_n(Nat.0)
        Nat.0.binom(Nat.0) = Nat.1
        (Nat.0 + Nat.0).binom(Nat.0.suc) = Nat.0.binom(Nat.1)
        choose_out_of_bounds(Nat.0, Nat.1)
        Nat.0 < Nat.1
        Nat.0.binom(Nat.1) = Nat.0
        Nat.2 * ((Nat.0 + Nat.0).binom(Nat.0) + (Nat.0 + Nat.0).binom(Nat.0.suc)) =
            Nat.2 * (Nat.1 + Nat.0)
        Nat.2 * (Nat.1 + Nat.0) = Nat.2
        (Nat.0.suc + Nat.0.suc).binom(Nat.0.suc) =
            Nat.2 * ((Nat.0 + Nat.0).binom(Nat.0) + (Nat.0 + Nat.0).binom(Nat.0.suc))
    } else {
        Nat.0 < n
        // pascal: binom(2n+2, n+1) = binom(2n+1, n) + binom(2n+1, n+1)
        pascal_suc_unbounded(n + n + Nat.1, n)
        (n + n + Nat.1).suc.binom(n.suc) =
            (n + n + Nat.1).binom(n) + (n + n + Nat.1).binom(n.suc)
        (n + n + Nat.1).suc = n.suc + n.suc
        choose_symm_sub_form(n + n + Nat.1, n.suc)
        nat_suc_lte_double_suc(n)
        n.suc <= n + n + Nat.1
        (n + n + Nat.1).binom(n.suc) = (n + n + Nat.1).binom(n + n + Nat.1 - n.suc)
        n + n + Nat.1 - n.suc = n
        (n + n + Nat.1).binom(n.suc) = (n + n + Nat.1).binom(n)
        (n.suc + n.suc).binom(n.suc) =
            Nat.2 * (n + n + Nat.1).binom(n)
        // pascal: binom(2n+1, n) = binom(2n, n-1) + binom(2n, n)
        pascal_suc_unbounded(n + n, n - Nat.1)
        (n + n).suc.binom((n - Nat.1).suc) =
            (n + n).binom(n - Nat.1) + (n + n).binom((n - Nat.1).suc)
        (n + n).suc = n + n + Nat.1
        nat_pred_suc(n)
        (n - Nat.1).suc = n
        (n + n + Nat.1).binom(n) =
            (n + n).binom(n - Nat.1) + (n + n).binom(n)
        choose_symm_sub_form(n + n, n - Nat.1)
        nat_pred_lte_double(n)
        n - Nat.1 <= n + n
        (n + n).binom(n - Nat.1) = (n + n).binom(n + n - (n - Nat.1))
        nat_double_sub_pred(n)
        n + n - (n - Nat.1) = n.suc
        (n + n).binom(n - Nat.1) = (n + n).binom(n.suc)
        (n + n + Nat.1).binom(n) =
            (n + n).binom(n.suc) + (n + n).binom(n)
        (n.suc + n.suc).binom(n.suc) =
            Nat.2 * ((n + n).binom(n.suc) + (n + n).binom(n))
        (n.suc + n.suc).binom(n.suc) =
            Nat.2 * ((n + n).binom(n) + (n + n).binom(n.suc))
    }
}

/// The adjacent binomial chain of length two:
/// `binom(2n + 2, n + 2) = binom(2n, n) + 2 * binom(2n, n + 1) + binom(2n, n + 2)`.
theorem central_binom_shift(n: Nat) {
    (n.suc + n.suc).binom(n.suc.suc) =
        (n + n).binom(n) + Nat.2 * (n + n).binom(n.suc) + (n + n).binom(n.suc.suc)
} by {
    pascal_suc_unbounded(n + n + Nat.1, n.suc)
    (n + n + Nat.1).suc.binom(n.suc.suc) =
        (n + n + Nat.1).binom(n.suc) + (n + n + Nat.1).binom(n.suc.suc)
    (n + n + Nat.1).suc = n.suc + n.suc
    choose_symm_sub_form(n + n + Nat.1, n.suc)
    nat_suc_lte_double_suc(n)
    n.suc <= n + n + Nat.1
    (n + n + Nat.1).binom(n.suc) = (n + n + Nat.1).binom(n + n + Nat.1 - n.suc)
    n + n + Nat.1 - n.suc = n
    (n + n + Nat.1).binom(n.suc) = (n + n + Nat.1).binom(n)
    pascal_suc_unbounded(n + n, n)
    (n + n).suc.binom(n.suc) =
        (n + n).binom(n) + (n + n).binom(n.suc)
    (n + n).suc = n + n + Nat.1
    (n + n + Nat.1).binom(n.suc) =
        (n + n).binom(n) + (n + n).binom(n.suc)
    pascal_suc_unbounded(n + n, n.suc)
    (n + n).suc.binom(n.suc.suc) =
        (n + n).binom(n.suc) + (n + n).binom(n.suc.suc)
    (n + n + Nat.1).binom(n.suc.suc) =
        (n + n).binom(n.suc) + (n + n).binom(n.suc.suc)
    (n.suc + n.suc).binom(n.suc.suc) =
        (n + n).binom(n) + (n + n).binom(n.suc) +
        (n + n).binom(n.suc) + (n + n).binom(n.suc.suc)
    (n + n).binom(n) + (n + n).binom(n.suc) +
        (n + n).binom(n.suc) + (n + n).binom(n.suc.suc) =
        (n + n).binom(n) + Nat.2 * (n + n).binom(n.suc) + (n + n).binom(n.suc.suc)
    (n.suc + n.suc).binom(n.suc.suc) =
        (n + n).binom(n) + Nat.2 * (n + n).binom(n.suc) + (n + n).binom(n.suc.suc)
}

/// The general Pascal chain: for `0 < j`,
/// `binom(2m + 2, m + 1 + j) = binom(2m, m + j - 1) + 2 * binom(2m, m + j) +
/// binom(2m, m + 1 + j)`.
theorem central_binom_step(m: Nat, j: Nat) {
    Nat.0 < j implies
        (m.suc + m.suc).binom(m.suc + j) =
            (m + m).binom(m + j - Nat.1) + Nat.2 * (m + m).binom(m + j) +
            (m + m).binom(m + j.suc)
} by {
    if Nat.0 < j {
        pascal_suc_unbounded(m + m + Nat.1, m + j)
        (m + m + Nat.1).suc.binom((m + j).suc) =
            (m + m + Nat.1).binom(m + j) + (m + m + Nat.1).binom((m + j).suc)
        (m + m + Nat.1).suc = m.suc + m.suc
        (m + j).suc = m + j.suc
        (m.suc + m.suc).binom(m + j.suc) =
            (m + m + Nat.1).binom(m + j) + (m + m + Nat.1).binom(m + j.suc)
        pascal_suc_unbounded(m + m, m + j - Nat.1)
        (m + m).suc.binom((m + j - Nat.1).suc) =
            (m + m).binom(m + j - Nat.1) + (m + m).binom((m + j - Nat.1).suc)
        (m + m).suc = m + m + Nat.1
        lt_imp_lte_suc(Nat.0, j)
        Nat.1 <= j
        lte_trans(Nat.1, j, m + j)
        Nat.1 <= m + j
        add_sub(m + j, Nat.1)
        m + j - Nat.1 + Nat.1 = m + j
        (m + j - Nat.1).suc = m + j - Nat.1 + Nat.1
        (m + j - Nat.1).suc = m + j
        (m + m + Nat.1).binom(m + j) =
            (m + m).binom(m + j - Nat.1) + (m + m).binom(m + j)
        pascal_suc_unbounded(m + m, m + j)
        (m + m).suc.binom((m + j).suc) =
            (m + m).binom(m + j) + (m + m).binom((m + j).suc)
        (m + m + Nat.1).binom(m + j.suc) =
            (m + m).binom(m + j) + (m + m).binom(m + j.suc)
        (m.suc + m.suc).binom(m + j.suc) =
            (m + m).binom(m + j - Nat.1) + (m + m).binom(m + j) +
            (m + m).binom(m + j) + (m + m).binom(m + j.suc)
        (m + m).binom(m + j - Nat.1) + (m + m).binom(m + j) +
            (m + m).binom(m + j) + (m + m).binom(m + j.suc) =
            (m + m).binom(m + j - Nat.1) + Nat.2 * (m + m).binom(m + j) +
            (m + m).binom(m + j.suc)
        (m.suc + m.suc).binom(m + j.suc) =
            (m + m).binom(m + j - Nat.1) + Nat.2 * (m + m).binom(m + j) +
            (m + m).binom(m + j.suc)
    }
}

// ---------------------------------------------------------------------------
// The closed form
// ---------------------------------------------------------------------------

/// The closed form: `C_n * (n + 1) = binom(2n, n)`.
theorem catalan_mul_suc(n: Nat) {
    catalan(n) * n.suc = (n + n).binom(n)
} by {
    if n = Nat.0 {
        catalan_zero
        catalan(Nat.0) = Nat.1
        Nat.0.suc = Nat.1
        catalan(Nat.0) * Nat.0.suc = Nat.1 * Nat.1
        Nat.1 * Nat.1 = Nat.1
        Nat.0 + Nat.0 = Nat.0
        (Nat.0 + Nat.0).binom(Nat.0) = Nat.1
        choose_n(Nat.0)
        catalan(Nat.0) * Nat.0.suc = (Nat.0 + Nat.0).binom(Nat.0)
    } else {
        Nat.0 < n
        central_binom_adjacent(n)
        n.suc * (n + n).binom(n.suc) = n * (n + n).binom(n)
        central_binom_adjacent_lte(n)
        (n + n).binom(n.suc) <= (n + n).binom(n)
        add_sub((n + n).binom(n), (n + n).binom(n.suc))
        (n + n).binom(n) - (n + n).binom(n.suc) + (n + n).binom(n.suc) =
            (n + n).binom(n)
        catalan(n) = (n + n).binom(n) - (n + n).binom(n.suc)
        // Step 1: (b - bp) * n.suc + n * b = b * n.suc
        distrib_right((n + n).binom(n) - (n + n).binom(n.suc), (n + n).binom(n.suc), n.suc)
        ((n + n).binom(n) - (n + n).binom(n.suc) + (n + n).binom(n.suc)) * n.suc =
            ((n + n).binom(n) - (n + n).binom(n.suc)) * n.suc +
            (n + n).binom(n.suc) * n.suc
        (n + n).binom(n) * n.suc =
            ((n + n).binom(n) - (n + n).binom(n.suc)) * n.suc +
            (n + n).binom(n.suc) * n.suc
        mul_comm((n + n).binom(n.suc), n.suc)
        (n + n).binom(n.suc) * n.suc = n.suc * (n + n).binom(n.suc)
        ((n + n).binom(n) - (n + n).binom(n.suc)) * n.suc + n.suc * (n + n).binom(n.suc) =
            ((n + n).binom(n) - (n + n).binom(n.suc)) * n.suc +
            (n + n).binom(n.suc) * n.suc
        (n + n).binom(n) * n.suc =
            ((n + n).binom(n) - (n + n).binom(n.suc)) * n.suc +
            n.suc * (n + n).binom(n.suc)
        (n + n).binom(n) * n.suc =
            ((n + n).binom(n) - (n + n).binom(n.suc)) * n.suc + n * (n + n).binom(n)
        // Step 2: b * n.suc - n * b = b
        mul_suc_right((n + n).binom(n), n)
        (n + n).binom(n) * n.suc = (n + n).binom(n) + (n + n).binom(n) * n
        mul_comm((n + n).binom(n), n)
        (n + n).binom(n) * n = n * (n + n).binom(n)
        (n + n).binom(n) * n.suc = (n + n).binom(n) + n * (n + n).binom(n)
        add_imp_sub_left(n * (n + n).binom(n), (n + n).binom(n),
            (n + n).binom(n) + n * (n + n).binom(n))
        n * (n + n).binom(n) + (n + n).binom(n) =
            (n + n).binom(n) + n * (n + n).binom(n)
        (n + n).binom(n) + n * (n + n).binom(n) - n * (n + n).binom(n) =
            (n + n).binom(n)
        (n + n).binom(n) * n.suc - n * (n + n).binom(n) = (n + n).binom(n)
        // Step 3: cancel n * b from both sides
        add_imp_sub(
            ((n + n).binom(n) - (n + n).binom(n.suc)) * n.suc,
            n * (n + n).binom(n),
            (n + n).binom(n) * n.suc)
        ((n + n).binom(n) - (n + n).binom(n.suc)) * n.suc +
            n * (n + n).binom(n) = (n + n).binom(n) * n.suc
        (n + n).binom(n) * n.suc - n * (n + n).binom(n) =
            ((n + n).binom(n) - (n + n).binom(n.suc)) * n.suc
        ((n + n).binom(n) - (n + n).binom(n.suc)) * n.suc = (n + n).binom(n)
        catalan(n) * n.suc = (n + n).binom(n)
    }
}

/// The shifted closed form: `C_{n+1} = binom(2n, n) - binom(2n, n + 2)`.
theorem catalan_shift(n: Nat) {
    catalan(n.suc) = (n + n).binom(n) - (n + n).binom(n.suc.suc)
} by {
    catalan(n.suc) = (n.suc + n.suc).binom(n.suc) - (n.suc + n.suc).binom(n.suc.suc)
    central_binom_suc(n)
    (n.suc + n.suc).binom(n.suc) =
        Nat.2 * ((n + n).binom(n) + (n + n).binom(n.suc))
    central_binom_shift(n)
    (n.suc + n.suc).binom(n.suc.suc) =
        (n + n).binom(n) + Nat.2 * (n + n).binom(n.suc) + (n + n).binom(n.suc.suc)
    catalan(n.suc) =
        Nat.2 * ((n + n).binom(n) + (n + n).binom(n.suc)) -
        ((n + n).binom(n) + Nat.2 * (n + n).binom(n.suc) + (n + n).binom(n.suc.suc))
    // Rewrite the two subtraction arguments into the cancelled shape.
    Nat.2 * ((n + n).binom(n) + (n + n).binom(n.suc)) =
        ((n + n).binom(n) + (n + n).binom(n)) + Nat.2 * (n + n).binom(n.suc)
    ((n + n).binom(n) + Nat.2 * (n + n).binom(n.suc) + (n + n).binom(n.suc.suc)) =
        ((n + n).binom(n) + (n + n).binom(n.suc.suc)) + Nat.2 * (n + n).binom(n.suc)
    catalan(n.suc) =
        (((n + n).binom(n) + (n + n).binom(n)) + Nat.2 * (n + n).binom(n.suc)) -
        (((n + n).binom(n) + (n + n).binom(n.suc.suc)) + Nat.2 * (n + n).binom(n.suc))
    nat_add_sub_cancel(
        (n + n).binom(n) + (n + n).binom(n),
        (n + n).binom(n) + (n + n).binom(n.suc.suc),
        Nat.2 * (n + n).binom(n.suc))
    (((n + n).binom(n) + (n + n).binom(n)) + Nat.2 * (n + n).binom(n.suc)) -
        (((n + n).binom(n) + (n + n).binom(n.suc.suc)) + Nat.2 * (n + n).binom(n.suc)) =
        ((n + n).binom(n) + (n + n).binom(n)) -
        ((n + n).binom(n) + (n + n).binom(n.suc.suc))
    catalan(n.suc) =
        ((n + n).binom(n) + (n + n).binom(n)) -
        ((n + n).binom(n) + (n + n).binom(n.suc.suc))
    nat_add_sub_cancel((n + n).binom(n), (n + n).binom(n.suc.suc), (n + n).binom(n))
    ((n + n).binom(n) + (n + n).binom(n)) -
        ((n + n).binom(n) + (n + n).binom(n.suc.suc)) =
        (n + n).binom(n) - (n + n).binom(n.suc.suc)
    catalan(n.suc) = (n + n).binom(n) - (n + n).binom(n.suc.suc)
}

// ---------------------------------------------------------------------------
// The convolution family
// ---------------------------------------------------------------------------

/// The `k`-th summand of the family sum
/// `T_n(j) = sum_{k=0}^{n} C_k * binom(2(n - k), n - k + j)`.
define cb_term(n: Nat, j: Nat, k: Nat) -> Nat {
    catalan(k) * ((n - k) + (n - k)).binom(n - k + j)
}

/// The family statement at `n`: `T_n(j) = binom(2n, n + j) + binom(2n, n + j + 1)`
/// for every `j`.
define catalan_family_hyp(n: Nat) -> Bool {
    forall(j: Nat) {
        partial(cb_term(n, j), n.suc) = (n + n).binom(n + j) + (n + n).binom(n + j.suc)
    }
}

/// The successor subtraction lemma for the shift `(n + 1) - k = (n - k) + 1`.
theorem nat_suc_sub_shift(n: Nat, k: Nat) {
    k <= n implies n.suc - k = (n - k) + Nat.1
} by {
    if k <= n {
        add_sub(n, k)
        n - k + k = n
        (n - k) + Nat.1 + k = n + Nat.1
        (n - k) + Nat.1 + k = k + ((n - k) + Nat.1)
        k + ((n - k) + Nat.1) = n + Nat.1
        add_imp_sub_left(k, (n - k) + Nat.1, n.suc)
        n.suc - k = (n - k) + Nat.1
    }
}

/// For b > 0, a + (b - 1) = a + b - 1.
theorem nat_add_sub_one(a: Nat, b: Nat) {
    Nat.0 < b implies a + (b - Nat.1) = a + b - Nat.1
} by {
    if Nat.0 < b {
        nat_pred_add_one(b)
        b - Nat.1 + Nat.1 = b
        (a + (b - Nat.1)) + Nat.1 = a + (b - Nat.1 + Nat.1)
        (a + (b - Nat.1)) + Nat.1 = a + b
        add_imp_sub_left(Nat.1, a + (b - Nat.1), a + b)
        a + b - Nat.1 = a + (b - Nat.1)
        a + (b - Nat.1) = a + b - Nat.1
    }
}

/// The base of the family induction at n = 0.
lemma catalan_family_zero {
    catalan_family_hyp(Nat.0)
} by {
    forall(j: Nat) {
        partial(cb_term(Nat.0, j), Nat.1) = cb_term(Nat.0, j, Nat.0)
        cb_term(Nat.0, j, Nat.0) =
            catalan(Nat.0) * ((Nat.0 - Nat.0) + (Nat.0 - Nat.0)).binom(Nat.0 - Nat.0 + j)
        Nat.0 - Nat.0 = Nat.0
        catalan_zero
        catalan(Nat.0) = Nat.1
        cb_term(Nat.0, j, Nat.0) = Nat.1 * Nat.0.binom(j)
        Nat.1 * Nat.0.binom(j) = Nat.0.binom(j)
        if j = Nat.0 {
            choose_n(Nat.0)
            Nat.0.binom(Nat.0) = Nat.1
            choose_out_of_bounds(Nat.0, Nat.1)
            Nat.0 < Nat.1
            Nat.0.binom(Nat.1) = Nat.0
            (Nat.0 + Nat.0).binom(Nat.0 + Nat.0) = Nat.0.binom(Nat.0)
            (Nat.0 + Nat.0).binom(Nat.0 + Nat.0.suc) = Nat.0.binom(Nat.0.suc)
            Nat.0 + Nat.0 = Nat.0
            Nat.0 + Nat.0.suc = Nat.1
            (Nat.0 + Nat.0).binom(Nat.0 + j) + (Nat.0 + Nat.0).binom(Nat.0 + j.suc) =
                Nat.0.binom(Nat.0) + Nat.0.binom(Nat.1)
            Nat.0.binom(Nat.0) + Nat.0.binom(Nat.1) = Nat.1 + Nat.0
            Nat.1 + Nat.0 = Nat.1
            partial(cb_term(Nat.0, j), Nat.1) =
                (Nat.0 + Nat.0).binom(Nat.0 + j) + (Nat.0 + Nat.0).binom(Nat.0 + j.suc)
        } else {
            Nat.0 < j
            choose_out_of_bounds(Nat.0, j)
            Nat.0.binom(j) = Nat.0
            lt_imp_lte_suc(Nat.0, j)
            Nat.1 <= j
            lte_trans(Nat.1, j, j.suc)
            Nat.1 <= j.suc
            choose_out_of_bounds(Nat.0, j.suc)
            Nat.0 < j.suc
            Nat.0.binom(j.suc) = Nat.0
            (Nat.0 + Nat.0).binom(Nat.0 + j) = Nat.0.binom(j)
            (Nat.0 + Nat.0).binom(Nat.0 + j.suc) = Nat.0.binom(j.suc)
            (Nat.0 + Nat.0).binom(Nat.0 + j) + (Nat.0 + Nat.0).binom(Nat.0 + j.suc) =
                Nat.0 + Nat.0
            Nat.0 + Nat.0 = Nat.0
            partial(cb_term(Nat.0, j), Nat.1) =
                (Nat.0 + Nat.0).binom(Nat.0 + j) + (Nat.0 + Nat.0).binom(Nat.0 + j.suc)
        }
    }
}

/// The middle sum for `0 < j`: rewriting `binom(2m + 2, m + 1 + j)` by the
/// Pascal chain splits the sum into the three neighbouring family sums.
theorem catalan_middle_suc(n: Nat, j: Nat) {
    Nat.0 < j implies
        partial(cb_term(n.suc, j), n) =
            partial(cb_term(n, j - Nat.1), n) +
            Nat.2 * partial(cb_term(n, j), n) +
            partial(cb_term(n, j.suc), n)
} by {
    if Nat.0 < j {
        forall(k: Nat) {
            if k < n {
                k <= n
                nat_suc_sub_shift(n, k)
                n.suc - k = (n - k) + Nat.1
                (n - k) + Nat.1 = (n - k).suc
                (n.suc - k) + (n.suc - k) = ((n - k).suc + (n - k).suc)
                n.suc - k + j = (n - k).suc + j
                central_binom_step(n - k, j)
                Nat.0 < j
                ((n - k).suc + (n - k).suc).binom((n - k).suc + j) =
                    ((n - k) + (n - k)).binom(n - k + j - Nat.1) +
                    Nat.2 * ((n - k) + (n - k)).binom(n - k + j) +
                    ((n - k) + (n - k)).binom(n - k + j.suc)
                cb_term(n.suc, j, k) =
                    catalan(k) * ((n.suc - k) + (n.suc - k)).binom(n.suc - k + j)
                cb_term(n.suc, j, k) =
                    catalan(k) *
                    (((n - k) + (n - k)).binom(n - k + j - Nat.1) +
                    Nat.2 * ((n - k) + (n - k)).binom(n - k + j) +
                    ((n - k) + (n - k)).binom(n - k + j.suc))
                distrib_left(catalan(k),
                    ((n - k) + (n - k)).binom(n - k + j - Nat.1),
                    Nat.2 * ((n - k) + (n - k)).binom(n - k + j) +
                        ((n - k) + (n - k)).binom(n - k + j.suc))
                catalan(k) *
                    (((n - k) + (n - k)).binom(n - k + j - Nat.1) +
                    Nat.2 * ((n - k) + (n - k)).binom(n - k + j) +
                    ((n - k) + (n - k)).binom(n - k + j.suc)) =
                    catalan(k) * ((n - k) + (n - k)).binom(n - k + j - Nat.1) +
                    catalan(k) *
                        (Nat.2 * ((n - k) + (n - k)).binom(n - k + j) +
                        ((n - k) + (n - k)).binom(n - k + j.suc))
                distrib_left(catalan(k),
                    Nat.2 * ((n - k) + (n - k)).binom(n - k + j),
                    ((n - k) + (n - k)).binom(n - k + j.suc))
                catalan(k) *
                    (Nat.2 * ((n - k) + (n - k)).binom(n - k + j) +
                    ((n - k) + (n - k)).binom(n - k + j.suc)) =
                    catalan(k) * (Nat.2 * ((n - k) + (n - k)).binom(n - k + j)) +
                    catalan(k) * ((n - k) + (n - k)).binom(n - k + j.suc)
                mul_assoc(catalan(k), Nat.2, ((n - k) + (n - k)).binom(n - k + j))
                catalan(k) * (Nat.2 * ((n - k) + (n - k)).binom(n - k + j)) =
                    (catalan(k) * Nat.2) * ((n - k) + (n - k)).binom(n - k + j)
                mul_comm(catalan(k), Nat.2)
                catalan(k) * Nat.2 = Nat.2 * catalan(k)
                (catalan(k) * Nat.2) * ((n - k) + (n - k)).binom(n - k + j) =
                    (Nat.2 * catalan(k)) * ((n - k) + (n - k)).binom(n - k + j)
                (Nat.2 * catalan(k)) * ((n - k) + (n - k)).binom(n - k + j) =
                    Nat.2 * (catalan(k) * ((n - k) + (n - k)).binom(n - k + j))
                catalan(k) * (Nat.2 * ((n - k) + (n - k)).binom(n - k + j)) =
                    Nat.2 * (catalan(k) * ((n - k) + (n - k)).binom(n - k + j))
                cb_term(n, j, k) =
                    catalan(k) * ((n - k) + (n - k)).binom(n - k + j)
                cb_term(n, j - Nat.1, k) =
                    catalan(k) * ((n - k) + (n - k)).binom(n - k + (j - Nat.1))
                nat_add_sub_one(n - k, j)
                Nat.0 < j
                (n - k) + (j - Nat.1) = n - k + j - Nat.1
                cb_term(n, j - Nat.1, k) =
                    catalan(k) * ((n - k) + (n - k)).binom(n - k + j - Nat.1)
                cb_term(n, j.suc, k) =
                    catalan(k) * ((n - k) + (n - k)).binom(n - k + j.suc)
                cb_term(n.suc, j, k) =
                    cb_term(n, j - Nat.1, k) + Nat.2 * cb_term(n, j, k) +
                    cb_term(n, j.suc, k)
                mul_fn(Nat.2, cb_term(n, j), k) = Nat.2 * cb_term(n, j, k)
                add_fn(mul_fn(Nat.2, cb_term(n, j)), cb_term(n, j.suc), k) =
                    mul_fn(Nat.2, cb_term(n, j), k) + cb_term(n, j.suc, k)
                add_fn(mul_fn(Nat.2, cb_term(n, j)), cb_term(n, j.suc), k) =
                    Nat.2 * cb_term(n, j, k) + cb_term(n, j.suc, k)
                add_fn(cb_term(n, j - Nat.1),
                    add_fn(mul_fn(Nat.2, cb_term(n, j)), cb_term(n, j.suc)), k) =
                    cb_term(n, j - Nat.1, k) +
                    add_fn(mul_fn(Nat.2, cb_term(n, j)), cb_term(n, j.suc), k)
                add_fn(cb_term(n, j - Nat.1),
                    add_fn(mul_fn(Nat.2, cb_term(n, j)), cb_term(n, j.suc)), k) =
                    cb_term(n, j - Nat.1, k) + Nat.2 * cb_term(n, j, k) +
                    cb_term(n, j.suc, k)
                cb_term(n.suc, j, k) =
                    add_fn(cb_term(n, j - Nat.1),
                        add_fn(mul_fn(Nat.2, cb_term(n, j)), cb_term(n, j.suc)), k)
            }
        }
        partial_pointwise_eq[Nat](cb_term(n.suc, j),
            add_fn(cb_term(n, j - Nat.1),
                add_fn(mul_fn(Nat.2, cb_term(n, j)), cb_term(n, j.suc))), n)
        partial(cb_term(n.suc, j), n) =
            partial(add_fn(cb_term(n, j - Nat.1),
                add_fn(mul_fn(Nat.2, cb_term(n, j)), cb_term(n, j.suc))), n)
        partial_add(cb_term(n, j - Nat.1),
            add_fn(mul_fn(Nat.2, cb_term(n, j)), cb_term(n, j.suc)), n)
        partial(cb_term(n, j - Nat.1), n) +
            partial(add_fn(mul_fn(Nat.2, cb_term(n, j)), cb_term(n, j.suc)), n) =
            partial(add_fn(cb_term(n, j - Nat.1),
                add_fn(mul_fn(Nat.2, cb_term(n, j)), cb_term(n, j.suc))), n)
        partial(cb_term(n, j - Nat.1), n) +
            partial(add_fn(mul_fn(Nat.2, cb_term(n, j)), cb_term(n, j.suc)), n) =
            partial(cb_term(n.suc, j), n)
        partial_add(mul_fn(Nat.2, cb_term(n, j)), cb_term(n, j.suc), n)
        partial(mul_fn(Nat.2, cb_term(n, j)), n) + partial(cb_term(n, j.suc), n) =
            partial(add_fn(mul_fn(Nat.2, cb_term(n, j)), cb_term(n, j.suc)), n)
        partial_scalar_mul[Nat](Nat.2, cb_term(n, j), n)
        Nat.2 * partial(cb_term(n, j), n) = partial(mul_fn(Nat.2, cb_term(n, j)), n)
        partial(cb_term(n, j - Nat.1), n) +
            (Nat.2 * partial(cb_term(n, j), n) + partial(cb_term(n, j.suc), n)) =
            partial(cb_term(n.suc, j), n)
        partial(cb_term(n, j - Nat.1), n) +
            Nat.2 * partial(cb_term(n, j), n) + partial(cb_term(n, j.suc), n) =
            partial(cb_term(n.suc, j), n)
        partial(cb_term(n.suc, j), n) =
            partial(cb_term(n, j - Nat.1), n) +
            Nat.2 * partial(cb_term(n, j), n) +
            partial(cb_term(n, j.suc), n)
    }
}

/// The middle sum for `j = 0`: the central binomial chain doubles the sum of
/// the `j = 0` and `j = 1` family sums.
theorem catalan_middle_zero(n: Nat) {
    partial(cb_term(n.suc, Nat.0), n) =
        Nat.2 * partial(cb_term(n, Nat.0), n) +
        Nat.2 * partial(cb_term(n, Nat.1), n)
} by {
    forall(k: Nat) {
        if k < n {
            k <= n
            nat_suc_sub_shift(n, k)
            n.suc - k = (n - k) + Nat.1
            (n - k) + Nat.1 = (n - k).suc
            (n.suc - k) + (n.suc - k) = ((n - k).suc + (n - k).suc)
            n.suc - k + Nat.0 = (n - k).suc + Nat.0
            central_binom_suc(n - k)
            ((n - k).suc + (n - k).suc).binom((n - k).suc) =
                Nat.2 * (((n - k) + (n - k)).binom(n - k) +
                    ((n - k) + (n - k)).binom((n - k).suc))
            (n - k).suc = n - k + Nat.1
            ((n - k) + (n - k)).binom((n - k).suc) =
                ((n - k) + (n - k)).binom(n - k + Nat.1)
            cb_term(n.suc, Nat.0, k) =
                catalan(k) * ((n.suc - k) + (n.suc - k)).binom(n.suc - k + Nat.0)
            n.suc - k + Nat.0 = n.suc - k
            cb_term(n.suc, Nat.0, k) =
                catalan(k) * ((n.suc - k) + (n.suc - k)).binom(n.suc - k)
            cb_term(n.suc, Nat.0, k) =
                catalan(k) * (Nat.2 * (((n - k) + (n - k)).binom(n - k) +
                    ((n - k) + (n - k)).binom(n - k + Nat.1)))
            Nat.2 * (((n - k) + (n - k)).binom(n - k) +
                ((n - k) + (n - k)).binom(n - k + Nat.1)) =
                ((n - k) + (n - k)).binom(n - k) +
                ((n - k) + (n - k)).binom(n - k) +
                Nat.2 * ((n - k) + (n - k)).binom(n - k + Nat.1)
            catalan(k) * (Nat.2 * (((n - k) + (n - k)).binom(n - k) +
                ((n - k) + (n - k)).binom(n - k + Nat.1))) =
                catalan(k) * (((n - k) + (n - k)).binom(n - k) +
                    ((n - k) + (n - k)).binom(n - k) +
                    Nat.2 * ((n - k) + (n - k)).binom(n - k + Nat.1))
            distrib_left(catalan(k),
                ((n - k) + (n - k)).binom(n - k),
                ((n - k) + (n - k)).binom(n - k) +
                    Nat.2 * ((n - k) + (n - k)).binom(n - k + Nat.1))
            catalan(k) * (((n - k) + (n - k)).binom(n - k) +
                ((n - k) + (n - k)).binom(n - k) +
                Nat.2 * ((n - k) + (n - k)).binom(n - k + Nat.1)) =
                catalan(k) * ((n - k) + (n - k)).binom(n - k) +
                catalan(k) * (((n - k) + (n - k)).binom(n - k) +
                    Nat.2 * ((n - k) + (n - k)).binom(n - k + Nat.1))
            distrib_left(catalan(k),
                ((n - k) + (n - k)).binom(n - k),
                Nat.2 * ((n - k) + (n - k)).binom(n - k + Nat.1))
            catalan(k) * (((n - k) + (n - k)).binom(n - k) +
                Nat.2 * ((n - k) + (n - k)).binom(n - k + Nat.1)) =
                catalan(k) * ((n - k) + (n - k)).binom(n - k) +
                catalan(k) * (Nat.2 * ((n - k) + (n - k)).binom(n - k + Nat.1))
            mul_assoc(catalan(k), Nat.2, ((n - k) + (n - k)).binom(n - k + Nat.1))
            catalan(k) * (Nat.2 * ((n - k) + (n - k)).binom(n - k + Nat.1)) =
                (catalan(k) * Nat.2) * ((n - k) + (n - k)).binom(n - k + Nat.1)
            mul_comm(catalan(k), Nat.2)
            catalan(k) * Nat.2 = Nat.2 * catalan(k)
            (catalan(k) * Nat.2) * ((n - k) + (n - k)).binom(n - k + Nat.1) =
                (Nat.2 * catalan(k)) * ((n - k) + (n - k)).binom(n - k + Nat.1)
            (Nat.2 * catalan(k)) * ((n - k) + (n - k)).binom(n - k + Nat.1) =
                Nat.2 * (catalan(k) * ((n - k) + (n - k)).binom(n - k + Nat.1))
            catalan(k) * (Nat.2 * ((n - k) + (n - k)).binom(n - k + Nat.1)) =
                Nat.2 * (catalan(k) * ((n - k) + (n - k)).binom(n - k + Nat.1))
            cb_term(n, Nat.0, k) =
                catalan(k) * ((n - k) + (n - k)).binom(n - k)
            cb_term(n, Nat.1, k) =
                catalan(k) * ((n - k) + (n - k)).binom(n - k + Nat.1)
            cb_term(n.suc, Nat.0, k) =
                cb_term(n, Nat.0, k) + cb_term(n, Nat.0, k) +
                Nat.2 * cb_term(n, Nat.1, k)
            cb_term(n.suc, Nat.0, k) =
                Nat.2 * cb_term(n, Nat.0, k) + Nat.2 * cb_term(n, Nat.1, k)
            mul_fn(Nat.2, cb_term(n, Nat.0), k) = Nat.2 * cb_term(n, Nat.0, k)
            mul_fn(Nat.2, cb_term(n, Nat.1), k) = Nat.2 * cb_term(n, Nat.1, k)
            add_fn(mul_fn(Nat.2, cb_term(n, Nat.0)), mul_fn(Nat.2, cb_term(n, Nat.1)), k) =
                Nat.2 * cb_term(n, Nat.0, k) + Nat.2 * cb_term(n, Nat.1, k)
            cb_term(n.suc, Nat.0, k) =
                add_fn(mul_fn(Nat.2, cb_term(n, Nat.0)), mul_fn(Nat.2, cb_term(n, Nat.1)), k)
        }
    }
    partial_pointwise_eq[Nat](cb_term(n.suc, Nat.0),
        add_fn(mul_fn(Nat.2, cb_term(n, Nat.0)), mul_fn(Nat.2, cb_term(n, Nat.1))), n)
    partial(cb_term(n.suc, Nat.0), n) =
        partial(add_fn(mul_fn(Nat.2, cb_term(n, Nat.0)), mul_fn(Nat.2, cb_term(n, Nat.1))), n)
    partial_add(mul_fn(Nat.2, cb_term(n, Nat.0)), mul_fn(Nat.2, cb_term(n, Nat.1)), n)
    partial(mul_fn(Nat.2, cb_term(n, Nat.0)), n) + partial(mul_fn(Nat.2, cb_term(n, Nat.1)), n) =
        partial(add_fn(mul_fn(Nat.2, cb_term(n, Nat.0)), mul_fn(Nat.2, cb_term(n, Nat.1))), n)
    partial(mul_fn(Nat.2, cb_term(n, Nat.0)), n) + partial(mul_fn(Nat.2, cb_term(n, Nat.1)), n) =
        partial(cb_term(n.suc, Nat.0), n)
    partial_scalar_mul[Nat](Nat.2, cb_term(n, Nat.0), n)
    Nat.2 * partial(cb_term(n, Nat.0), n) = partial(mul_fn(Nat.2, cb_term(n, Nat.0)), n)
    partial_scalar_mul[Nat](Nat.2, cb_term(n, Nat.1), n)
    Nat.2 * partial(cb_term(n, Nat.1), n) = partial(mul_fn(Nat.2, cb_term(n, Nat.1)), n)
    Nat.2 * partial(cb_term(n, Nat.0), n) + Nat.2 * partial(cb_term(n, Nat.1), n) =
        partial(cb_term(n.suc, Nat.0), n)
    partial(cb_term(n.suc, Nat.0), n) =
        Nat.2 * partial(cb_term(n, Nat.0), n) + Nat.2 * partial(cb_term(n, Nat.1), n)
}
/// The target of the family step expands by the two Pascal chains into the
/// fully expanded six-term form shared with the induction side.

/// The target of the family step expands by the two Pascal chains into the
/// fully expanded six-term form shared with the induction side.
lemma catalan_target_expand(n: Nat, j: Nat) {
    Nat.0 < j implies
        (n.suc + n.suc).binom(n.suc + j) + (n.suc + n.suc).binom(n.suc + j.suc) =
            (n + n).binom(n + j - Nat.1) + (n + n).binom(n + j) + (n + n).binom(n + j) +
            (n + n).binom(n + j) + (n + n).binom(n + j.suc) + (n + n).binom(n + j.suc) +
            (n + n).binom(n + j.suc) + (n + n).binom(n + j.suc.suc)
} by {
    if Nat.0 < j {
        central_binom_step(n, j)
        (n.suc + n.suc).binom(n.suc + j) =
            (n + n).binom(n + j - Nat.1) + Nat.2 * (n + n).binom(n + j) +
            (n + n).binom(n + j.suc)
        lt_imp_lt_suc(Nat.0, j)
        Nat.0 < j.suc
        central_binom_step(n, j.suc)
        (n.suc + n.suc).binom(n.suc + j.suc) =
            (n + n).binom(n + j.suc - Nat.1) + Nat.2 * (n + n).binom(n + j.suc) +
            (n + n).binom(n + j.suc.suc)
        lt_imp_lte_suc(Nat.0, j)
        Nat.1 <= j
        lte_add_right(j, Nat.0, n)
        Nat.0 <= n
        Nat.0 + j <= n + j
        Nat.0 + j = j
        j <= n + j
        lt_and_lte(Nat.0, j, n + j)
        Nat.0 < j and j <= n + j
        Nat.0 < n + j
        nat_pred_add_one(n + j)
        n + j - Nat.1 + Nat.1 = n + j
        n + j.suc - Nat.1 = n + j
        (n.suc + n.suc).binom(n.suc + j.suc) =
            (n + n).binom(n + j) + Nat.2 * (n + n).binom(n + j.suc) +
            (n + n).binom(n + j.suc.suc)
        // congruence on the first summand
        (n.suc + n.suc).binom(n.suc + j) + (n.suc + n.suc).binom(n.suc + j.suc) =
            ((n + n).binom(n + j - Nat.1) + Nat.2 * (n + n).binom(n + j) +
            (n + n).binom(n + j.suc)) + (n.suc + n.suc).binom(n.suc + j.suc)
        // congruence on the second summand
        ((n + n).binom(n + j - Nat.1) + Nat.2 * (n + n).binom(n + j) +
            (n + n).binom(n + j.suc)) + (n.suc + n.suc).binom(n.suc + j.suc) =
            ((n + n).binom(n + j - Nat.1) + Nat.2 * (n + n).binom(n + j) +
            (n + n).binom(n + j.suc)) +
            ((n + n).binom(n + j) + Nat.2 * (n + n).binom(n + j.suc) +
            (n + n).binom(n + j.suc.suc))
        // flatten
        ((n + n).binom(n + j - Nat.1) + Nat.2 * (n + n).binom(n + j) +
            (n + n).binom(n + j.suc)) +
            ((n + n).binom(n + j) + Nat.2 * (n + n).binom(n + j.suc) +
            (n + n).binom(n + j.suc.suc)) =
            (n + n).binom(n + j - Nat.1) + Nat.2 * (n + n).binom(n + j) +
            (n + n).binom(n + j.suc) + (n + n).binom(n + j) +
            Nat.2 * (n + n).binom(n + j.suc) + (n + n).binom(n + j.suc.suc)
        // expand 2 * x in place
        mul_two_left((n + n).binom(n + j))
        Nat.2 * (n + n).binom(n + j) = (n + n).binom(n + j) + (n + n).binom(n + j)
        mul_two_left((n + n).binom(n + j.suc))
        Nat.2 * (n + n).binom(n + j.suc) = (n + n).binom(n + j.suc) + (n + n).binom(n + j.suc)
        (n + n).binom(n + j - Nat.1) + Nat.2 * (n + n).binom(n + j) +
            (n + n).binom(n + j.suc) + (n + n).binom(n + j) +
            Nat.2 * (n + n).binom(n + j.suc) + (n + n).binom(n + j.suc.suc) =
            (n + n).binom(n + j - Nat.1) + (n + n).binom(n + j) + (n + n).binom(n + j) +
            (n + n).binom(n + j.suc) + (n + n).binom(n + j) + (n + n).binom(n + j.suc) +
            (n + n).binom(n + j.suc) + (n + n).binom(n + j.suc.suc)
        // swap the middle C and B: C + B = B + C
        add_comm((n + n).binom(n + j.suc), (n + n).binom(n + j))
        (n + n).binom(n + j.suc) + (n + n).binom(n + j) =
            (n + n).binom(n + j) + (n + n).binom(n + j.suc)
        (n + n).binom(n + j - Nat.1) + (n + n).binom(n + j) + (n + n).binom(n + j) +
            (n + n).binom(n + j.suc) + (n + n).binom(n + j) + (n + n).binom(n + j.suc) +
            (n + n).binom(n + j.suc) + (n + n).binom(n + j.suc.suc) =
            (n + n).binom(n + j - Nat.1) + (n + n).binom(n + j) + (n + n).binom(n + j) +
            (n + n).binom(n + j) + (n + n).binom(n + j.suc) + (n + n).binom(n + j.suc) +
            (n + n).binom(n + j.suc) + (n + n).binom(n + j.suc.suc)
        (n.suc + n.suc).binom(n.suc + j) + (n.suc + n.suc).binom(n.suc + j.suc) =
            (n + n).binom(n + j - Nat.1) + (n + n).binom(n + j) + (n + n).binom(n + j) +
            (n + n).binom(n + j) + (n + n).binom(n + j.suc) + (n + n).binom(n + j.suc) +
            (n + n).binom(n + j.suc) + (n + n).binom(n + j.suc.suc)
    }
}

/// binom(2n, n + 2) is at most binom(2n, n).
theorem central_binom_second_lte(n: Nat) {
    (n + n).binom(n.suc.suc) <= (n + n).binom(n)
} by {
    lte_ref(n)
    n <= n
    lte_suc_suc(n, n)
    n <= n implies n.suc <= n.suc
    n.suc <= n.suc
    lte_trans(n, n.suc, n.suc)
    n <= n.suc
    lte_add_right(n, n, n.suc)
    n <= n.suc implies n + n <= n + n.suc
    n + n <= n + n.suc
    n + n.suc = n.suc + n
    n + n <= n.suc + n
    lte_add_right(n.suc, n, n.suc)
    n <= n.suc implies n.suc + n <= n.suc + n.suc
    n + n <= n.suc + n.suc
    Nat.2 * n.suc = n.suc + n.suc
    n + n <= Nat.2 * n.suc
    binom_suc_le_binom(n + n, n.suc)
    (n + n).binom(n.suc.suc) <= (n + n).binom(n.suc)
    central_binom_adjacent_lte(n)
    (n + n).binom(n.suc) <= (n + n).binom(n)
    lte_trans((n + n).binom(n.suc.suc), (n + n).binom(n.suc), (n + n).binom(n))
    (n + n).binom(n.suc.suc) <= (n + n).binom(n)
}

/// The j = 0 algebra: C_{n+1} + 2 * binom(2n, n + 2) = binom(2n, n) + binom(2n, n + 2).
theorem catalan_shift_merge(n: Nat) {
    catalan(n.suc) + Nat.2 * (n + n).binom(n.suc.suc) =
        (n + n).binom(n) + (n + n).binom(n.suc.suc)
} by {
    catalan_shift(n)
    catalan(n.suc) = (n + n).binom(n) - (n + n).binom(n.suc.suc)
    central_binom_second_lte(n)
    (n + n).binom(n.suc.suc) <= (n + n).binom(n)
    add_sub((n + n).binom(n), (n + n).binom(n.suc.suc))
    (n + n).binom(n) - (n + n).binom(n.suc.suc) + (n + n).binom(n.suc.suc) =
        (n + n).binom(n)
    catalan(n.suc) + (n + n).binom(n.suc.suc) =
        (n + n).binom(n)
    catalan(n.suc) + (n + n).binom(n.suc.suc) + (n + n).binom(n.suc.suc) =
        (n + n).binom(n) + (n + n).binom(n.suc.suc)
    catalan(n.suc) + Nat.2 * (n + n).binom(n.suc.suc) =
        (n + n).binom(n) + (n + n).binom(n.suc.suc)
}

/// The j = 0 family-step algebra:
/// 2 * binom(2n, n) + 4 * binom(2n, n + 1) + 2 * binom(2n, n + 2) + C_{n+1}
/// equals 3 * binom(2n, n) + 4 * binom(2n, n + 1) + binom(2n, n + 2).
theorem catalan_j0_cancel(n: Nat) {
    Nat.2 * (n + n).binom(n) + Nat.4 * (n + n).binom(n.suc) +
        Nat.2 * (n + n).binom(n.suc.suc) + catalan(n.suc) =
        Nat.3 * (n + n).binom(n) + Nat.4 * (n + n).binom(n.suc) +
        (n + n).binom(n.suc.suc)
} by {
    // move 2 b2 to the front, then swap the first two atoms
    add_comm(Nat.2 * (n + n).binom(n) + Nat.4 * (n + n).binom(n.suc),
        Nat.2 * (n + n).binom(n.suc.suc))
    Nat.2 * (n + n).binom(n) + Nat.4 * (n + n).binom(n.suc) +
        Nat.2 * (n + n).binom(n.suc.suc) =
        Nat.2 * (n + n).binom(n.suc.suc) +
        (Nat.2 * (n + n).binom(n) + Nat.4 * (n + n).binom(n.suc))
    add_comm(Nat.2 * (n + n).binom(n.suc.suc),
        Nat.2 * (n + n).binom(n))
    Nat.2 * (n + n).binom(n.suc.suc) + Nat.2 * (n + n).binom(n) =
        Nat.2 * (n + n).binom(n) + Nat.2 * (n + n).binom(n.suc.suc)
    Nat.2 * (n + n).binom(n.suc.suc) +
        (Nat.2 * (n + n).binom(n) + Nat.4 * (n + n).binom(n.suc)) =
        Nat.2 * (n + n).binom(n) + Nat.2 * (n + n).binom(n.suc.suc) +
        Nat.4 * (n + n).binom(n.suc)
    // 2 b2 + c = b0 + b2, from catalan_shift_merge
    catalan_shift_merge(n)
    catalan(n.suc) + Nat.2 * (n + n).binom(n.suc.suc) =
        (n + n).binom(n) + (n + n).binom(n.suc.suc)
    Nat.2 * (n + n).binom(n.suc.suc) + catalan(n.suc) =
        catalan(n.suc) + Nat.2 * (n + n).binom(n.suc.suc)
    Nat.2 * (n + n).binom(n.suc.suc) + catalan(n.suc) =
        (n + n).binom(n) + (n + n).binom(n.suc.suc)
    // the sum is 2 b0 + 4 b1 + (2 b2 + c)
    Nat.2 * (n + n).binom(n) + Nat.2 * (n + n).binom(n.suc.suc) +
        Nat.4 * (n + n).binom(n.suc) + catalan(n.suc) =
        Nat.2 * (n + n).binom(n) + Nat.4 * (n + n).binom(n.suc) +
        (Nat.2 * (n + n).binom(n.suc.suc) + catalan(n.suc))
    Nat.2 * (n + n).binom(n) + Nat.4 * (n + n).binom(n.suc) +
        (Nat.2 * (n + n).binom(n.suc.suc) + catalan(n.suc)) =
        Nat.2 * (n + n).binom(n) + Nat.4 * (n + n).binom(n.suc) +
        ((n + n).binom(n) + (n + n).binom(n.suc.suc))
    // move the plain b0 to the front of the four atoms
    Nat.2 * (n + n).binom(n) + Nat.4 * (n + n).binom(n.suc) +
        ((n + n).binom(n) + (n + n).binom(n.suc.suc)) =
        (n + n).binom(n) + Nat.2 * (n + n).binom(n) + Nat.4 * (n + n).binom(n.suc) +
        (n + n).binom(n.suc.suc)
    // b0 + 2 b0 = 3 b0
    (n + n).binom(n) + Nat.2 * (n + n).binom(n) =
        Nat.3 * (n + n).binom(n)
    (n + n).binom(n) + Nat.2 * (n + n).binom(n) + Nat.4 * (n + n).binom(n.suc) +
        (n + n).binom(n.suc.suc) =
        Nat.3 * (n + n).binom(n) + Nat.4 * (n + n).binom(n.suc) +
        (n + n).binom(n.suc.suc)
    Nat.2 * (n + n).binom(n) + Nat.4 * (n + n).binom(n.suc) +
        ((n + n).binom(n) + (n + n).binom(n.suc.suc)) =
        Nat.3 * (n + n).binom(n) + Nat.4 * (n + n).binom(n.suc) +
        (n + n).binom(n.suc.suc)
    Nat.2 * (n + n).binom(n) + Nat.4 * (n + n).binom(n.suc) +
        Nat.2 * (n + n).binom(n.suc.suc) + catalan(n.suc) =
        Nat.3 * (n + n).binom(n) + Nat.4 * (n + n).binom(n.suc) +
        (n + n).binom(n.suc.suc)
}

/// The j = 0 endpoint merge: the sum
/// 2 * partial(cb_term(n, 0), n) + 2 * partial(cb_term(n, 1), n) + 2 * C_n
/// equals 2 * partial(cb_term(n, 0), n.suc) + 2 * partial(cb_term(n, 1), n.suc).
theorem catalan_j0_merge(n: Nat) {
    Nat.2 * partial(cb_term(n, Nat.0), n) +
        Nat.2 * partial(cb_term(n, Nat.1), n) + Nat.2 * catalan(n) =
        Nat.2 * partial(cb_term(n, Nat.0), n.suc) +
        Nat.2 * partial(cb_term(n, Nat.1), n.suc)
} by {
    partial_split_last(cb_term(n, Nat.0), n)
    partial(cb_term(n, Nat.0), n.suc) =
        partial(cb_term(n, Nat.0), n) + cb_term(n, Nat.0, n)
    cb_term(n, Nat.0, n) =
        catalan(n) * ((n - n) + (n - n)).binom(n - n + Nat.0)
    n - n = Nat.0
    cb_term(n, Nat.0, n) = catalan(n) * Nat.0.binom(Nat.0)
    choose_n(Nat.0)
    Nat.0.binom(Nat.0) = Nat.1
    cb_term(n, Nat.0, n) = catalan(n)
    partial(cb_term(n, Nat.0), n.suc) =
        partial(cb_term(n, Nat.0), n) + catalan(n)
    partial_split_last(cb_term(n, Nat.1), n)
    partial(cb_term(n, Nat.1), n.suc) =
        partial(cb_term(n, Nat.1), n) + cb_term(n, Nat.1, n)
    cb_term(n, Nat.1, n) =
        catalan(n) * ((n - n) + (n - n)).binom(n - n + Nat.1)
    choose_out_of_bounds(Nat.0, Nat.1)
    Nat.0 < Nat.1
    Nat.0.binom(Nat.1) = Nat.0
    cb_term(n, Nat.1, n) = catalan(n) * Nat.0
    cb_term(n, Nat.1, n) = Nat.0
    partial(cb_term(n, Nat.1), n.suc) = partial(cb_term(n, Nat.1), n)
    // 2 P0 + 2 P1 + 2 c = 2 (P0 + c) + 2 P1
    // move 2 c to the front, then swap the first two atoms
    add_comm(Nat.2 * partial(cb_term(n, Nat.0), n) +
        Nat.2 * partial(cb_term(n, Nat.1), n), Nat.2 * catalan(n))
    Nat.2 * partial(cb_term(n, Nat.0), n) +
        Nat.2 * partial(cb_term(n, Nat.1), n) + Nat.2 * catalan(n) =
        Nat.2 * catalan(n) + Nat.2 * partial(cb_term(n, Nat.0), n) +
        Nat.2 * partial(cb_term(n, Nat.1), n)
    add_comm(Nat.2 * catalan(n), Nat.2 * partial(cb_term(n, Nat.0), n))
    Nat.2 * catalan(n) + Nat.2 * partial(cb_term(n, Nat.0), n) =
        Nat.2 * partial(cb_term(n, Nat.0), n) + Nat.2 * catalan(n)
    Nat.2 * catalan(n) + Nat.2 * partial(cb_term(n, Nat.0), n) +
        Nat.2 * partial(cb_term(n, Nat.1), n) =
        Nat.2 * partial(cb_term(n, Nat.0), n) + Nat.2 * catalan(n) +
        Nat.2 * partial(cb_term(n, Nat.1), n)
    // 2 P0 + 2 c = 2 (P0 + c)
    distrib_left(Nat.2, partial(cb_term(n, Nat.0), n), catalan(n))
    Nat.2 * (partial(cb_term(n, Nat.0), n) + catalan(n)) =
        Nat.2 * partial(cb_term(n, Nat.0), n) + Nat.2 * catalan(n)
    Nat.2 * partial(cb_term(n, Nat.0), n) + Nat.2 * catalan(n) +
        Nat.2 * partial(cb_term(n, Nat.1), n) =
        Nat.2 * (partial(cb_term(n, Nat.0), n) + catalan(n)) +
        Nat.2 * partial(cb_term(n, Nat.1), n)
    // P0 + c = T(n, 0)
    partial(cb_term(n, Nat.0), n) + catalan(n) =
        partial(cb_term(n, Nat.0), n.suc)
    Nat.2 * (partial(cb_term(n, Nat.0), n) + catalan(n)) +
        Nat.2 * partial(cb_term(n, Nat.1), n) =
        Nat.2 * partial(cb_term(n, Nat.0), n.suc) +
        Nat.2 * partial(cb_term(n, Nat.1), n)
    Nat.2 * partial(cb_term(n, Nat.0), n.suc) +
        Nat.2 * partial(cb_term(n, Nat.1), n) =
        Nat.2 * partial(cb_term(n, Nat.0), n.suc) +
        Nat.2 * partial(cb_term(n, Nat.1), n.suc)
    Nat.2 * partial(cb_term(n, Nat.0), n) +
        Nat.2 * partial(cb_term(n, Nat.1), n) + Nat.2 * catalan(n) =
        Nat.2 * partial(cb_term(n, Nat.0), n.suc) +
        Nat.2 * partial(cb_term(n, Nat.1), n.suc)
}

/// The family induction step at index `0 < j`.
lemma catalan_family_suc_pos(n: Nat, j: Nat) {
    Nat.0 < j and catalan_family_hyp(n) implies
        partial(cb_term(n.suc, j), n.suc.suc) =
            (n.suc + n.suc).binom(n.suc + j) + (n.suc + n.suc).binom(n.suc + j.suc)
} by {
    if Nat.0 < j and catalan_family_hyp(n) {
        if forall(i: Nat) {
            partial(cb_term(n, i), n.suc) = (n + n).binom(n + i) + (n + n).binom(n + i.suc)
        } {
            // Split the sum into the k < n part and the two endpoint terms.
            partial_split_last(cb_term(n.suc, j), n.suc)
            partial(cb_term(n.suc, j), n.suc.suc) =
                partial(cb_term(n.suc, j), n.suc) + cb_term(n.suc, j, n.suc)
            partial_split_last(cb_term(n.suc, j), n)
            partial(cb_term(n.suc, j), n.suc) =
                partial(cb_term(n.suc, j), n) + cb_term(n.suc, j, n)
            partial(cb_term(n.suc, j), n.suc.suc) =
                partial(cb_term(n.suc, j), n) + cb_term(n.suc, j, n) +
                cb_term(n.suc, j, n.suc)
            // The k < n part splits by the Pascal chain.
            catalan_middle_suc(n, j)
            Nat.0 < j
            partial(cb_term(n.suc, j), n) =
                partial(cb_term(n, j - Nat.1), n) +
                Nat.2 * partial(cb_term(n, j), n) +
                partial(cb_term(n, j.suc), n)
            partial(cb_term(n.suc, j), n.suc.suc) =
                partial(cb_term(n, j - Nat.1), n) +
                Nat.2 * partial(cb_term(n, j), n) +
                partial(cb_term(n, j.suc), n) +
                cb_term(n.suc, j, n) + cb_term(n.suc, j, n.suc)
            // The endpoint at k = n + 1 vanishes.
            cb_term(n.suc, j, n.suc) =
                catalan(n.suc) * ((n.suc - n.suc) + (n.suc - n.suc)).binom(n.suc - n.suc + j)
            n.suc - n.suc = Nat.0
            cb_term(n.suc, j, n.suc) = catalan(n.suc) * (Nat.0 + Nat.0).binom(Nat.0 + j)
            choose_out_of_bounds(Nat.0, j)
            Nat.0 < j
            Nat.0.binom(j) = Nat.0
            cb_term(n.suc, j, n.suc) = catalan(n.suc) * Nat.0
            catalan(n.suc) * Nat.0 = Nat.0
            cb_term(n.suc, j, n.suc) = Nat.0
            partial(cb_term(n.suc, j), n.suc.suc) =
                partial(cb_term(n, j - Nat.1), n) +
                Nat.2 * partial(cb_term(n, j), n) +
                partial(cb_term(n, j.suc), n) + cb_term(n.suc, j, n)
            // The endpoint at k = n is catalan(n) * binom(2, 1 + j).
            cb_term(n.suc, j, n) =
                catalan(n) * ((n.suc - n) + (n.suc - n)).binom(n.suc - n + j)
            n.suc - n = Nat.1
            cb_term(n.suc, j, n) = catalan(n) * (Nat.1 + Nat.1).binom(Nat.1 + j)
            Nat.1 + Nat.1 = Nat.2
            cb_term(n.suc, j, n) = catalan(n) * Nat.2.binom(Nat.1 + j)
            partial(cb_term(n.suc, j), n.suc.suc) =
                partial(cb_term(n, j - Nat.1), n) +
                Nat.2 * partial(cb_term(n, j), n) +
                partial(cb_term(n, j.suc), n) +
                catalan(n) * Nat.2.binom(Nat.1 + j)
            // The family sums T(n, i) = partial(cb_term(n, i), n.suc) split off
            // their k = n endpoint.
            partial_split_last(cb_term(n, j - Nat.1), n)
            partial(cb_term(n, j - Nat.1), n.suc) =
                partial(cb_term(n, j - Nat.1), n) + cb_term(n, j - Nat.1, n)
            partial_split_last(cb_term(n, j), n)
            partial(cb_term(n, j), n.suc) =
                partial(cb_term(n, j), n) + cb_term(n, j, n)
            partial_split_last(cb_term(n, j.suc), n)
            partial(cb_term(n, j.suc), n.suc) =
                partial(cb_term(n, j.suc), n) + cb_term(n, j.suc, n)
            cb_term(n, j, n) =
                catalan(n) * ((n - n) + (n - n)).binom(n - n + j)
            n - n = Nat.0
            cb_term(n, j, n) = catalan(n) * (Nat.0 + Nat.0).binom(j)
            choose_out_of_bounds(Nat.0, j)
            Nat.0.binom(j) = Nat.0
            cb_term(n, j, n) = catalan(n) * Nat.0
            cb_term(n, j, n) = Nat.0
            cb_term(n, j.suc, n) =
                catalan(n) * ((n - n) + (n - n)).binom(n - n + j.suc)
            choose_out_of_bounds(Nat.0, j.suc)
            Nat.0 < j.suc
            Nat.0.binom(j.suc) = Nat.0
            cb_term(n, j.suc, n) = catalan(n) * Nat.0
            cb_term(n, j.suc, n) = Nat.0
            partial(cb_term(n, j), n) = partial(cb_term(n, j), n.suc)
            partial(cb_term(n, j.suc), n) = partial(cb_term(n, j.suc), n.suc)
            if j = Nat.1 {
                // For j = 1 the j - 1 family sum keeps its k = n endpoint
                // catalan(n) * binom(0, 0) = catalan(n), which merges with the
                // endpoint catalan(n) * binom(2, 2) = catalan(n).
                cb_term(n, j - Nat.1, n) =
                    catalan(n) * ((n - n) + (n - n)).binom(n - n + (j - Nat.1))
                n - n = Nat.0
                j - Nat.1 = Nat.0
                cb_term(n, j - Nat.1, n) = catalan(n) * Nat.0.binom(Nat.0)
                choose_n(Nat.0)
                Nat.0.binom(Nat.0) = Nat.1
                cb_term(n, j - Nat.1, n) = catalan(n)
                partial(cb_term(n, j - Nat.1), n.suc) =
                    partial(cb_term(n, j - Nat.1), n) + catalan(n)
                Nat.2.binom(Nat.1 + j) = Nat.2.binom(Nat.2)
                Nat.1 + j = Nat.2
                choose_n(Nat.2)
                Nat.2.binom(Nat.2) = Nat.1
                catalan(n) * Nat.2.binom(Nat.1 + j) = catalan(n)
                // Reassemble: (P0 + 2 P1 + P2) + c = (P0 + c) + 2 P1 + P2.
                partial(cb_term(n.suc, j), n.suc.suc) =
                    partial(cb_term(n, j - Nat.1), n) +
                    Nat.2 * partial(cb_term(n, j), n) +
                    partial(cb_term(n, j.suc), n) + catalan(n)
                add_comm(partial(cb_term(n, j - Nat.1), n) +
                    Nat.2 * partial(cb_term(n, j), n) + partial(cb_term(n, j.suc), n),
                    catalan(n))
                partial(cb_term(n.suc, j), n.suc.suc) =
                    catalan(n) + partial(cb_term(n, j - Nat.1), n) +
                    Nat.2 * partial(cb_term(n, j), n) + partial(cb_term(n, j.suc), n)
                add_comm(catalan(n), partial(cb_term(n, j - Nat.1), n))
                catalan(n) + partial(cb_term(n, j - Nat.1), n) =
                    partial(cb_term(n, j - Nat.1), n) + catalan(n)
                partial(cb_term(n.suc, j), n.suc.suc) =
                    partial(cb_term(n, j - Nat.1), n) + catalan(n) +
                    Nat.2 * partial(cb_term(n, j), n) + partial(cb_term(n, j.suc), n)
                // P0 + c = T(n, 0), P1 = T(n, 1), P2 = T(n, 2).
                partial(cb_term(n.suc, j), n.suc.suc) =
                    partial(cb_term(n, j - Nat.1), n.suc) +
                    Nat.2 * partial(cb_term(n, j), n) + partial(cb_term(n, j.suc), n)
                partial(cb_term(n.suc, j), n.suc.suc) =
                    partial(cb_term(n, j - Nat.1), n.suc) +
                    Nat.2 * partial(cb_term(n, j), n.suc) + partial(cb_term(n, j.suc), n.suc)
            } else {
                // For j >= 2 all endpoint corrections vanish.
                j != Nat.1
                Nat.0 < j
                lt_or_lte(Nat.1, j)
                Nat.1 <= j
                Nat.1 != j
                Nat.1 < j
                choose_out_of_bounds(Nat.0, j - Nat.1)
                Nat.0 < j - Nat.1
                Nat.0.binom(j - Nat.1) = Nat.0
                cb_term(n, j - Nat.1, n) =
                    catalan(n) * ((n - n) + (n - n)).binom(n - n + (j - Nat.1))
                n - n = Nat.0
                cb_term(n, j - Nat.1, n) = catalan(n) * Nat.0.binom(j - Nat.1)
                cb_term(n, j - Nat.1, n) = catalan(n) * Nat.0
                cb_term(n, j - Nat.1, n) = Nat.0
                partial(cb_term(n, j - Nat.1), n) = partial(cb_term(n, j - Nat.1), n.suc)
                lt_add_left(Nat.1, Nat.1, j)
                Nat.1 < j implies Nat.1 + Nat.1 < Nat.1 + j
                Nat.1 + Nat.1 < Nat.1 + j
                Nat.1 + Nat.1 = Nat.2
                Nat.2 < Nat.1 + j
                choose_out_of_bounds(Nat.2, Nat.1 + j)
                Nat.2.binom(Nat.1 + j) = Nat.0
                catalan(n) * Nat.2.binom(Nat.1 + j) = catalan(n) * Nat.0
                catalan(n) * Nat.2.binom(Nat.1 + j) = Nat.0
                partial(cb_term(n.suc, j), n.suc.suc) =
                    partial(cb_term(n, j - Nat.1), n) +
                    Nat.2 * partial(cb_term(n, j), n) +
                    partial(cb_term(n, j.suc), n)
                partial(cb_term(n.suc, j), n.suc.suc) =
                    partial(cb_term(n, j - Nat.1), n.suc) +
                    Nat.2 * partial(cb_term(n, j), n.suc) +
                    partial(cb_term(n, j.suc), n.suc)
            }
            // The induction hypothesis at j - 1, j and j + 1, with the
            // index normalizations for j - 1.
            partial(cb_term(n, j - Nat.1), n.suc) =
                (n + n).binom(n + (j - Nat.1)) + (n + n).binom(n + (j - Nat.1).suc)
            nat_pred_suc(j)
            Nat.0 < j
            (j - Nat.1).suc = j
            n + (j - Nat.1).suc = n + j
            nat_add_sub_one(n, j)
            Nat.0 < j
            n + (j - Nat.1) = n + j - Nat.1
            partial(cb_term(n, j - Nat.1), n.suc) =
                (n + n).binom(n + j - Nat.1) + (n + n).binom(n + j)
            partial(cb_term(n, j), n.suc) =
                (n + n).binom(n + j) + (n + n).binom(n + j.suc)
            partial(cb_term(n, j.suc), n.suc) =
                (n + n).binom(n + j.suc) + (n + n).binom(n + j.suc.suc)
            partial(cb_term(n.suc, j), n.suc.suc) =
                (n + n).binom(n + j - Nat.1) + (n + n).binom(n + j) +
                Nat.2 * ((n + n).binom(n + j) + (n + n).binom(n + j.suc)) +
                (n + n).binom(n + j.suc) + (n + n).binom(n + j.suc.suc)
            partial(cb_term(n.suc, j), n.suc.suc) =
                (n + n).binom(n + j - Nat.1) + (n + n).binom(n + j) +
                Nat.2 * (n + n).binom(n + j) + Nat.2 * (n + n).binom(n + j.suc) +
                (n + n).binom(n + j.suc) + (n + n).binom(n + j.suc.suc)
            // Expand 2 * x in place.
            mul_two_left((n + n).binom(n + j))
            Nat.2 * (n + n).binom(n + j) = (n + n).binom(n + j) + (n + n).binom(n + j)
            mul_two_left((n + n).binom(n + j.suc))
            Nat.2 * (n + n).binom(n + j.suc) = (n + n).binom(n + j.suc) + (n + n).binom(n + j.suc)
            partial(cb_term(n.suc, j), n.suc.suc) =
                (n + n).binom(n + j - Nat.1) + (n + n).binom(n + j) + (n + n).binom(n + j) +
                (n + n).binom(n + j) + (n + n).binom(n + j.suc) + (n + n).binom(n + j.suc) +
                (n + n).binom(n + j.suc) + (n + n).binom(n + j.suc.suc)
            // The target expands to the same form.
            catalan_target_expand(n, j)
            Nat.0 < j
            (n.suc + n.suc).binom(n.suc + j) + (n.suc + n.suc).binom(n.suc + j.suc) =
                (n + n).binom(n + j - Nat.1) + (n + n).binom(n + j) + (n + n).binom(n + j) +
                (n + n).binom(n + j) + (n + n).binom(n + j.suc) + (n + n).binom(n + j.suc) +
                (n + n).binom(n + j.suc) + (n + n).binom(n + j.suc.suc)
            partial(cb_term(n.suc, j), n.suc.suc) =
                (n.suc + n.suc).binom(n.suc + j) + (n.suc + n.suc).binom(n.suc + j.suc)
        }
        if not forall(i: Nat) {
            partial(cb_term(n, i), n.suc) = (n + n).binom(n + i) + (n + n).binom(n + i.suc)
        } {
            false
        }
        partial(cb_term(n.suc, j), n.suc.suc) =
            (n.suc + n.suc).binom(n.suc + j) + (n.suc + n.suc).binom(n.suc + j.suc)
    }
}

/// The family induction step at `j = 0`.
lemma catalan_family_suc_zero(n: Nat) {
    catalan_family_hyp(n) implies
        partial(cb_term(n.suc, Nat.0), n.suc.suc) =
            (n.suc + n.suc).binom(n.suc) + (n.suc + n.suc).binom(n.suc.suc)
} by {
    if catalan_family_hyp(n) {
        if forall(i: Nat) {
            partial(cb_term(n, i), n.suc) = (n + n).binom(n + i) + (n + n).binom(n + i.suc)
        } {
            // Split the sum and apply the j = 0 middle lemma.
            partial_split_last(cb_term(n.suc, Nat.0), n.suc)
            partial(cb_term(n.suc, Nat.0), n.suc.suc) =
                partial(cb_term(n.suc, Nat.0), n.suc) + cb_term(n.suc, Nat.0, n.suc)
            partial_split_last(cb_term(n.suc, Nat.0), n)
            partial(cb_term(n.suc, Nat.0), n.suc) =
                partial(cb_term(n.suc, Nat.0), n) + cb_term(n.suc, Nat.0, n)
            partial(cb_term(n.suc, Nat.0), n.suc.suc) =
                partial(cb_term(n.suc, Nat.0), n) + cb_term(n.suc, Nat.0, n) +
                cb_term(n.suc, Nat.0, n.suc)
            catalan_middle_zero(n)
            partial(cb_term(n.suc, Nat.0), n) =
                Nat.2 * partial(cb_term(n, Nat.0), n) +
                Nat.2 * partial(cb_term(n, Nat.1), n)
            partial(cb_term(n.suc, Nat.0), n.suc.suc) =
                Nat.2 * partial(cb_term(n, Nat.0), n) +
                Nat.2 * partial(cb_term(n, Nat.1), n) +
                cb_term(n.suc, Nat.0, n) + cb_term(n.suc, Nat.0, n.suc)
            // The endpoints are 2 * C_n and C_{n+1}.
            cb_term(n.suc, Nat.0, n) =
                catalan(n) * ((n.suc - n) + (n.suc - n)).binom(n.suc - n + Nat.0)
            n.suc - n = Nat.1
            cb_term(n.suc, Nat.0, n) = catalan(n) * (Nat.1 + Nat.1).binom(Nat.1)
            Nat.1 + Nat.1 = Nat.2
            cb_term(n.suc, Nat.0, n) = catalan(n) * Nat.2.binom(Nat.1)
            choose_one(Nat.2)
            Nat.2.binom(Nat.1) = Nat.2
            cb_term(n.suc, Nat.0, n) = catalan(n) * Nat.2
            mul_comm(catalan(n), Nat.2)
            catalan(n) * Nat.2 = Nat.2 * catalan(n)
            cb_term(n.suc, Nat.0, n) = Nat.2 * catalan(n)
            cb_term(n.suc, Nat.0, n.suc) =
                catalan(n.suc) * ((n.suc - n.suc) + (n.suc - n.suc)).binom(n.suc - n.suc + Nat.0)
            n.suc - n.suc = Nat.0
            cb_term(n.suc, Nat.0, n.suc) = catalan(n.suc) * Nat.0.binom(Nat.0)
            choose_n(Nat.0)
            Nat.0.binom(Nat.0) = Nat.1
            cb_term(n.suc, Nat.0, n.suc) = catalan(n.suc)
            partial(cb_term(n.suc, Nat.0), n.suc.suc) =
                Nat.2 * partial(cb_term(n, Nat.0), n) +
                Nat.2 * partial(cb_term(n, Nat.1), n) +
                Nat.2 * catalan(n) + catalan(n.suc)
            // Merge the endpoints into the family sums.
            catalan_j0_merge(n)
            partial(cb_term(n.suc, Nat.0), n.suc.suc) =
                Nat.2 * partial(cb_term(n, Nat.0), n.suc) +
                Nat.2 * partial(cb_term(n, Nat.1), n.suc) + catalan(n.suc)
            // The induction hypothesis at j = 0 and j = 1, with the
            // index normalization for j = 1.
            partial(cb_term(n, Nat.0), n.suc) =
                (n + n).binom(n) + (n + n).binom(n.suc)
            add_one_right(n)
            n + Nat.1 = n.suc
            add_suc_right(n, Nat.1)
            n + Nat.1.suc = (n + Nat.1).suc
            (n + Nat.1).suc = n.suc.suc
            partial(cb_term(n, Nat.1), n.suc) =
                (n + n).binom(n + Nat.1) + (n + n).binom(n + Nat.1.suc)
            partial(cb_term(n, Nat.1), n.suc) =
                (n + n).binom(n.suc) + (n + n).binom(n.suc.suc)
            partial(cb_term(n.suc, Nat.0), n.suc.suc) =
                Nat.2 * ((n + n).binom(n) + (n + n).binom(n.suc)) +
                Nat.2 * ((n + n).binom(n.suc) + (n + n).binom(n.suc.suc)) +
                catalan(n.suc)
            distrib_left(Nat.2, (n + n).binom(n), (n + n).binom(n.suc))
            Nat.2 * ((n + n).binom(n) + (n + n).binom(n.suc)) =
                Nat.2 * (n + n).binom(n) + Nat.2 * (n + n).binom(n.suc)
            distrib_left(Nat.2, (n + n).binom(n.suc), (n + n).binom(n.suc.suc))
            Nat.2 * ((n + n).binom(n.suc) + (n + n).binom(n.suc.suc)) =
                Nat.2 * (n + n).binom(n.suc) + Nat.2 * (n + n).binom(n.suc.suc)
            partial(cb_term(n.suc, Nat.0), n.suc.suc) =
                Nat.2 * (n + n).binom(n) + Nat.2 * (n + n).binom(n.suc) +
                Nat.2 * (n + n).binom(n.suc) + Nat.2 * (n + n).binom(n.suc.suc) +
                catalan(n.suc)
            distrib_right(Nat.2, Nat.2, (n + n).binom(n.suc))
            (Nat.2 + Nat.2) * (n + n).binom(n.suc) =
                Nat.2 * (n + n).binom(n.suc) + Nat.2 * (n + n).binom(n.suc)
            Nat.2 + Nat.2 = Nat.4
            (Nat.2 + Nat.2) * (n + n).binom(n.suc) = Nat.4 * (n + n).binom(n.suc)
            Nat.2 * (n + n).binom(n.suc) + Nat.2 * (n + n).binom(n.suc) =
                Nat.4 * (n + n).binom(n.suc)
            partial(cb_term(n.suc, Nat.0), n.suc.suc) =
                Nat.2 * (n + n).binom(n) + Nat.4 * (n + n).binom(n.suc) +
                Nat.2 * (n + n).binom(n.suc.suc) + catalan(n.suc)
            // The j = 0 algebra: 2 b0 + 4 b1 + 2 b2 + C_{n+1} = 3 b0 + 4 b1 + b2.
            catalan_j0_cancel(n)
            partial(cb_term(n.suc, Nat.0), n.suc.suc) =
                Nat.3 * (n + n).binom(n) + Nat.4 * (n + n).binom(n.suc) +
                (n + n).binom(n.suc.suc)
            // The target expands to the same form.
            central_binom_suc(n)
            (n.suc + n.suc).binom(n.suc) =
                Nat.2 * (n + n).binom(n) + Nat.2 * (n + n).binom(n.suc)
            central_binom_shift(n)
            (n.suc + n.suc).binom(n.suc.suc) =
                (n + n).binom(n) + Nat.2 * (n + n).binom(n.suc) + (n + n).binom(n.suc.suc)
            (n.suc + n.suc).binom(n.suc) + (n.suc + n.suc).binom(n.suc.suc) =
                (Nat.2 * (n + n).binom(n) + Nat.2 * (n + n).binom(n.suc)) +
                (n.suc + n.suc).binom(n.suc.suc)
            (Nat.2 * (n + n).binom(n) + Nat.2 * (n + n).binom(n.suc)) +
                (n.suc + n.suc).binom(n.suc.suc) =
                (Nat.2 * (n + n).binom(n) + Nat.2 * (n + n).binom(n.suc)) +
                ((n + n).binom(n) + Nat.2 * (n + n).binom(n.suc) + (n + n).binom(n.suc.suc))
            add_comm_4(Nat.2 * (n + n).binom(n), Nat.2 * (n + n).binom(n.suc),
                (n + n).binom(n), Nat.2 * (n + n).binom(n.suc))
            (Nat.2 * (n + n).binom(n) + Nat.2 * (n + n).binom(n.suc)) +
                ((n + n).binom(n) + Nat.2 * (n + n).binom(n.suc)) =
                (Nat.2 * (n + n).binom(n) + (n + n).binom(n)) +
                (Nat.2 * (n + n).binom(n.suc) + Nat.2 * (n + n).binom(n.suc))
            (Nat.2 * (n + n).binom(n) + Nat.2 * (n + n).binom(n.suc)) +
                ((n + n).binom(n) + Nat.2 * (n + n).binom(n.suc) + (n + n).binom(n.suc.suc)) =
                (Nat.2 * (n + n).binom(n) + (n + n).binom(n)) +
                (Nat.2 * (n + n).binom(n.suc) + Nat.2 * (n + n).binom(n.suc)) +
                (n + n).binom(n.suc.suc)
            Nat.2 * (n + n).binom(n) + (n + n).binom(n) =
                Nat.3 * (n + n).binom(n)
            distrib_right(Nat.2, Nat.2, (n + n).binom(n.suc))
            (Nat.2 + Nat.2) * (n + n).binom(n.suc) =
                Nat.2 * (n + n).binom(n.suc) + Nat.2 * (n + n).binom(n.suc)
            Nat.2 + Nat.2 = Nat.4
            (Nat.2 + Nat.2) * (n + n).binom(n.suc) = Nat.4 * (n + n).binom(n.suc)
            Nat.2 * (n + n).binom(n.suc) + Nat.2 * (n + n).binom(n.suc) =
                Nat.4 * (n + n).binom(n.suc)
            (Nat.2 * (n + n).binom(n) + (n + n).binom(n)) +
                (Nat.2 * (n + n).binom(n.suc) + Nat.2 * (n + n).binom(n.suc)) +
                (n + n).binom(n.suc.suc) =
                Nat.3 * (n + n).binom(n) + Nat.4 * (n + n).binom(n.suc) +
                (n + n).binom(n.suc.suc)
            (n.suc + n.suc).binom(n.suc) + (n.suc + n.suc).binom(n.suc.suc) =
                Nat.3 * (n + n).binom(n) + Nat.4 * (n + n).binom(n.suc) +
                (n + n).binom(n.suc.suc)
            partial(cb_term(n.suc, Nat.0), n.suc.suc) =
                (n.suc + n.suc).binom(n.suc) + (n.suc + n.suc).binom(n.suc.suc)
        }
        if not forall(i: Nat) {
            partial(cb_term(n, i), n.suc) = (n + n).binom(n + i) + (n + n).binom(n + i.suc)
        } {
            false
        }
        partial(cb_term(n.suc, Nat.0), n.suc.suc) =
            (n.suc + n.suc).binom(n.suc) + (n.suc + n.suc).binom(n.suc.suc)
    }
}

/// The successor step of the family induction.
lemma catalan_family_suc(n: Nat) {
    catalan_family_hyp(n) implies catalan_family_hyp(n.suc)
} by {
    if catalan_family_hyp(n) {
        if forall(j: Nat) {
            partial(cb_term(n, j), n.suc) = (n + n).binom(n + j) + (n + n).binom(n + j.suc)
        } {
            forall(j: Nat) {
                if j = Nat.0 {
                    catalan_family_suc_zero(n)
                    catalan_family_hyp(n)
                    partial(cb_term(n.suc, j), n.suc.suc) =
                        (n.suc + n.suc).binom(n.suc + j) + (n.suc + n.suc).binom(n.suc + j.suc)
                } else {
                    Nat.0 < j
                    catalan_family_suc_pos(n, j)
                    Nat.0 < j and catalan_family_hyp(n)
                    partial(cb_term(n.suc, j), n.suc.suc) =
                        (n.suc + n.suc).binom(n.suc + j) + (n.suc + n.suc).binom(n.suc + j.suc)
                }
            }
        }
        if not forall(j: Nat) {
            partial(cb_term(n, j), n.suc) = (n + n).binom(n + j) + (n + n).binom(n + j.suc)
        } {
            false
        }
        catalan_family_hyp(n.suc)
    }
}

/// The convolution family: for every n and j,
/// `sum_{k=0}^{n} C_k * binom(2(n - k), n - k + j) =
/// binom(2n, n + j) + binom(2n, n + j + 1)`.
theorem catalan_conv_family(n: Nat) {
    catalan_family_hyp(n)
} by {
    define p(m: Nat) -> Bool {
        catalan_family_hyp(m)
    }
    catalan_family_zero
    p(Nat.0)
    forall(m: Nat) {
        if p(m) {
            catalan_family_suc(m)
            catalan_family_hyp(m) implies catalan_family_hyp(m.suc)
            p(m.suc)
        }
    }
    alt_induction(p)
    forall(m: Nat) { p(m) }
    catalan_family_hyp(n)
}

/// The `k`-th summand of the Catalan convolution sum.
define catalan_conv_term(n: Nat, k: Nat) -> Nat {
    catalan(k) * catalan(n - k)
}

/// The first-moment summand: (k + 1) * C_k * C_{n-k}.
define catalan_first_sum_term(n: Nat, k: Nat) -> Nat {
    k.suc * catalan_conv_term(n, k)
}

/// The weighted summand: (n - k + 1) * C_k * C_{n-k}.
define catalan_weighted_term(n: Nat, k: Nat) -> Nat {
    (n - k).suc * catalan_conv_term(n, k)
}

/// The k-th summand of the family sum at j = 0 equals the weighted summand:
/// C_k * binom(2(n - k), n - k) = (n - k + 1) * C_k * C_{n-k}.
theorem catalan_cb_weighted(n: Nat, k: Nat) {
    cb_term(n, Nat.0, k) = catalan_weighted_term(n, k)
} by {
    cb_term(n, Nat.0, k) =
        catalan(k) * ((n - k) + (n - k)).binom(n - k + Nat.0)
    n - k + Nat.0 = n - k
    cb_term(n, Nat.0, k) =
        catalan(k) * ((n - k) + (n - k)).binom(n - k)
    catalan_mul_suc(n - k)
    catalan(n - k) * (n - k).suc = ((n - k) + (n - k)).binom(n - k)
    cb_term(n, Nat.0, k) = catalan(k) * (catalan(n - k) * (n - k).suc)
    mul_assoc(catalan(k), catalan(n - k), (n - k).suc)
    (catalan(k) * catalan(n - k)) * (n - k).suc =
        catalan(k) * (catalan(n - k) * (n - k).suc)
    cb_term(n, Nat.0, k) = (catalan(k) * catalan(n - k)) * (n - k).suc
    catalan_conv_term(n, k) = catalan(k) * catalan(n - k)
    mul_comm(catalan(k) * catalan(n - k), (n - k).suc)
    (catalan(k) * catalan(n - k)) * (n - k).suc =
        (n - k).suc * (catalan(k) * catalan(n - k))
    cb_term(n, Nat.0, k) = (n - k).suc * (catalan(k) * catalan(n - k))
    cb_term(n, Nat.0, k) = (n - k).suc * catalan_conv_term(n, k)
    cb_term(n, Nat.0, k) = catalan_weighted_term(n, k)
}

/// The sum of the weighted summands equals the family sum at j = 0.
theorem catalan_weighted_sum(n: Nat) {
    partial(catalan_weighted_term(n), n.suc) =
        (n + n).binom(n) + (n + n).binom(n.suc)
} by {
    catalan_conv_family(n)
    catalan_family_hyp(n)
    if forall(i: Nat) {
        partial(cb_term(n, i), n.suc) = (n + n).binom(n + i) + (n + n).binom(n + i.suc)
    } {
        partial(cb_term(n, Nat.0), n.suc) = (n + n).binom(n) + (n + n).binom(n.suc)
        forall(k: Nat) {
            if k < n.suc {
                catalan_cb_weighted(n, k)
                cb_term(n, Nat.0, k) = catalan_weighted_term(n, k)
            }
        }
        partial_pointwise_eq[Nat](cb_term(n, Nat.0), catalan_weighted_term(n), n.suc)
        partial(cb_term(n, Nat.0), n.suc) = partial(catalan_weighted_term(n), n.suc)
        partial(catalan_weighted_term(n), n.suc) =
            (n + n).binom(n) + (n + n).binom(n.suc)
    }
    if not forall(i: Nat) {
        partial(cb_term(n, i), n.suc) = (n + n).binom(n + i) + (n + n).binom(n + i.suc)
    } {
        false
    }
    partial(catalan_weighted_term(n), n.suc) =
        (n + n).binom(n) + (n + n).binom(n.suc)
}

/// The k-th summand of the split convolution: the (n + 2)-scaled term is the
/// sum of the first-moment and the weighted summands.
theorem catalan_split_term(n: Nat, k: Nat) {
    k <= n implies
        (n + Nat.2) * catalan_conv_term(n, k) =
            catalan_first_sum_term(n, k) + catalan_weighted_term(n, k)
} by {
    if k <= n {
        add_sub(n, k)
        n - k + k = n
        k + (n - k) = n
        (k + Nat.1) + (n - k + Nat.1) = (k + (n - k)) + Nat.2
        k + (n - k) + Nat.2 = n + Nat.2
        (k + Nat.1) + (n - k + Nat.1) = n + Nat.2
        distrib_right(k + Nat.1, n - k + Nat.1, catalan_conv_term(n, k))
    (k + Nat.1 + (n - k + Nat.1)) * catalan_conv_term(n, k) =
        (k + Nat.1) * catalan_conv_term(n, k) +
        (n - k + Nat.1) * catalan_conv_term(n, k)
    (k + Nat.1 + (n - k + Nat.1)) * catalan_conv_term(n, k) =
        (n + Nat.2) * catalan_conv_term(n, k)
    (n + Nat.2) * catalan_conv_term(n, k) =
        (k + Nat.1) * catalan_conv_term(n, k) +
        (n - k + Nat.1) * catalan_conv_term(n, k)
    k.suc = k + Nat.1
    catalan_first_sum_term(n, k) = k.suc * catalan_conv_term(n, k)
    catalan_first_sum_term(n, k) = (k + Nat.1) * catalan_conv_term(n, k)
    catalan_weighted_term(n, k) = (n - k).suc * catalan_conv_term(n, k)
    (n - k).suc = n - k + Nat.1
    catalan_weighted_term(n, k) = (n - k + Nat.1) * catalan_conv_term(n, k)
        (n + Nat.2) * catalan_conv_term(n, k) =
            catalan_first_sum_term(n, k) + catalan_weighted_term(n, k)
    }
}

/// The first-moment summand equals the reversed weighted summand:
/// (k + 1) * C_k * C_{n-k} = C_{n-k} * binom(2k, k) at the reversed index.
theorem catalan_first_reversed(n: Nat, k: Nat) {
    k <= n implies
        reverse_index(catalan_first_sum_term(n), n, k) =
            catalan(k) * ((n - k) + (n - k)).binom(n - k)
} by {
    if k <= n {
        reverse_index(catalan_first_sum_term(n), n, k) =
            catalan_first_sum_term(n, n - k)
        catalan_first_sum_term(n, n - k) = (n - k).suc * catalan_conv_term(n, n - k)
        catalan_conv_term(n, n - k) = catalan(n - k) * catalan(n - (n - k))
        add_sub(n, k)
        n - k + k = n
        add_imp_sub(n - k, k, n)
        n - (n - k) = k
        catalan_conv_term(n, n - k) = catalan(n - k) * catalan(k)
        catalan_first_sum_term(n, n - k) = (n - k).suc * (catalan(n - k) * catalan(k))
        mul_assoc((n - k).suc, catalan(n - k), catalan(k))
        ((n - k).suc * catalan(n - k)) * catalan(k) =
            (n - k).suc * (catalan(n - k) * catalan(k))
        catalan_first_sum_term(n, n - k) =
            ((n - k).suc * catalan(n - k)) * catalan(k)
        catalan_mul_suc(n - k)
        catalan(n - k) * (n - k).suc = ((n - k) + (n - k)).binom(n - k)
        mul_comm((n - k).suc, catalan(n - k))
        (n - k).suc * catalan(n - k) = catalan(n - k) * (n - k).suc
        (n - k).suc * catalan(n - k) = ((n - k) + (n - k)).binom(n - k)
        catalan_first_sum_term(n, n - k) =
            ((n - k) + (n - k)).binom(n - k) * catalan(k)
        mul_comm(((n - k) + (n - k)).binom(n - k), catalan(k))
        ((n - k) + (n - k)).binom(n - k) * catalan(k) =
            catalan(k) * ((n - k) + (n - k)).binom(n - k)
        catalan_first_sum_term(n, n - k) =
            catalan(k) * ((n - k) + (n - k)).binom(n - k)
        reverse_index(catalan_first_sum_term(n), n, k) =
            catalan(k) * ((n - k) + (n - k)).binom(n - k)
    }
}

/// The sum of the first-moment summands equals the family sum at j = 0.
theorem catalan_first_sum(n: Nat) {
    partial(catalan_first_sum_term(n), n.suc) =
        (n + n).binom(n) + (n + n).binom(n.suc)
} by {
    partial_reverse[Nat](catalan_first_sum_term(n), n)
    partial(catalan_first_sum_term(n), n.suc) =
        partial(reverse_index(catalan_first_sum_term(n), n), n.suc)
    forall(k: Nat) {
        if k < n.suc {
            lt_imp_lte_suc(k, n)
            k <= n
            catalan_first_reversed(n, k)
            reverse_index(catalan_first_sum_term(n), n, k) =
                catalan(k) * ((n - k) + (n - k)).binom(n - k)
            cb_term(n, Nat.0, k) =
                catalan(k) * ((n - k) + (n - k)).binom(n - k + Nat.0)
            n - k + Nat.0 = n - k
            cb_term(n, Nat.0, k) =
                catalan(k) * ((n - k) + (n - k)).binom(n - k)
            reverse_index(catalan_first_sum_term(n), n, k) = cb_term(n, Nat.0, k)
        }
    }
    partial_pointwise_eq[Nat](reverse_index(catalan_first_sum_term(n), n), cb_term(n, Nat.0), n.suc)
    partial(reverse_index(catalan_first_sum_term(n), n), n.suc) =
        partial(cb_term(n, Nat.0), n.suc)
    catalan_conv_family(n)
    catalan_family_hyp(n)
    if forall(i: Nat) {
        partial(cb_term(n, i), n.suc) = (n + n).binom(n + i) + (n + n).binom(n + i.suc)
    } {
        partial(cb_term(n, Nat.0), n.suc) = (n + n).binom(n) + (n + n).binom(n.suc)
    }
    if not forall(i: Nat) {
        partial(cb_term(n, i), n.suc) = (n + n).binom(n + i) + (n + n).binom(n + i.suc)
    } {
        false
    }
    partial(cb_term(n, Nat.0), n.suc) = (n + n).binom(n) + (n + n).binom(n.suc)
    partial(catalan_first_sum_term(n), n.suc) =
        (n + n).binom(n) + (n + n).binom(n.suc)
}

/// The Catalan recurrence: `C_{n+1} = sum_{k=0}^{n} C_k * C_{n-k}`.
theorem catalan_recurrence(n: Nat) {
    catalan(n.suc) = partial(catalan_conv_term(n), n.suc)
} by {
    // S(n) * (n + 2) = sum_k (n + 2) * C_k * C_{n-k}
    partial_scalar_mul[Nat](n + Nat.2, catalan_conv_term(n), n.suc)
    (n + Nat.2) * partial(catalan_conv_term(n), n.suc) =
        partial(mul_fn(n + Nat.2, catalan_conv_term(n)), n.suc)
    mul_comm(n + Nat.2, partial(catalan_conv_term(n), n.suc))
    (n + Nat.2) * partial(catalan_conv_term(n), n.suc) =
        partial(catalan_conv_term(n), n.suc) * (n + Nat.2)
    partial(catalan_conv_term(n), n.suc) * (n + Nat.2) =
        partial(mul_fn(n + Nat.2, catalan_conv_term(n)), n.suc)
    // each summand splits into the first-moment and weighted parts
    forall(k: Nat) {
        if k < n.suc {
            lt_imp_lte_suc(k, n)
            k <= n
            catalan_split_term(n, k)
            (n + Nat.2) * catalan_conv_term(n, k) =
                catalan_first_sum_term(n, k) + catalan_weighted_term(n, k)
            mul_fn(n + Nat.2, catalan_conv_term(n), k) =
                (n + Nat.2) * catalan_conv_term(n, k)
            mul_fn(n + Nat.2, catalan_conv_term(n), k) =
                catalan_first_sum_term(n, k) + catalan_weighted_term(n, k)
            add_fn(catalan_first_sum_term(n), catalan_weighted_term(n), k) =
                catalan_first_sum_term(n, k) + catalan_weighted_term(n, k)
            mul_fn(n + Nat.2, catalan_conv_term(n), k) =
                add_fn(catalan_first_sum_term(n), catalan_weighted_term(n), k)
        }
    }
    partial_pointwise_eq[Nat](mul_fn(n + Nat.2, catalan_conv_term(n)),
        add_fn(catalan_first_sum_term(n), catalan_weighted_term(n)), n.suc)
    partial(mul_fn(n + Nat.2, catalan_conv_term(n)), n.suc) =
        partial(add_fn(catalan_first_sum_term(n), catalan_weighted_term(n)), n.suc)
    partial_add(catalan_first_sum_term(n), catalan_weighted_term(n), n.suc)
    partial(catalan_first_sum_term(n), n.suc) +
        partial(catalan_weighted_term(n), n.suc) =
        partial(add_fn(catalan_first_sum_term(n), catalan_weighted_term(n)), n.suc)
    partial(catalan_conv_term(n), n.suc) * (n + Nat.2) =
        partial(catalan_first_sum_term(n), n.suc) +
        partial(catalan_weighted_term(n), n.suc)
    // both sums equal T(n, 0) = b_n + b'_n
    catalan_first_sum(n)
    partial(catalan_first_sum_term(n), n.suc) =
        (n + n).binom(n) + (n + n).binom(n.suc)
    catalan_weighted_sum(n)
    partial(catalan_weighted_term(n), n.suc) =
        (n + n).binom(n) + (n + n).binom(n.suc)
    partial(catalan_conv_term(n), n.suc) * (n + Nat.2) =
        (n + n).binom(n) + (n + n).binom(n.suc) +
        (n + n).binom(n) + (n + n).binom(n.suc)
    (n + n).binom(n) + (n + n).binom(n.suc) +
        (n + n).binom(n) + (n + n).binom(n.suc) =
        Nat.2 * ((n + n).binom(n) + (n + n).binom(n.suc))
    partial(catalan_conv_term(n), n.suc) * (n + Nat.2) =
        Nat.2 * ((n + n).binom(n) + (n + n).binom(n.suc))
    central_binom_suc(n)
    Nat.2 * ((n + n).binom(n) + (n + n).binom(n.suc)) =
        (n.suc + n.suc).binom(n.suc)
    partial(catalan_conv_term(n), n.suc) * (n + Nat.2) =
        (n.suc + n.suc).binom(n.suc)
    catalan_mul_suc(n.suc)
    catalan(n.suc) * n.suc.suc = (n.suc + n.suc).binom(n.suc)
    n.suc.suc = n + Nat.2
    catalan(n.suc) * (n + Nat.2) = (n.suc + n.suc).binom(n.suc)
    partial(catalan_conv_term(n), n.suc) * (n + Nat.2) =
        catalan(n.suc) * (n + Nat.2)
    mul_cancel_left(n + Nat.2, partial(catalan_conv_term(n), n.suc), catalan(n.suc))
    n + Nat.2 != Nat.0
    (n + Nat.2) * partial(catalan_conv_term(n), n.suc) =
        (n + Nat.2) * catalan(n.suc)
    partial(catalan_conv_term(n), n.suc) = catalan(n.suc)
    catalan(n.suc) = partial(catalan_conv_term(n), n.suc)
}

// ---------------------------------------------------------------------------
// Interpretations (documented, not formalized here)
// ---------------------------------------------------------------------------
//
// The Catalan number C_n counts, among many other things, the number of Dyck
// words of length 2n: sequences of n up-steps and n down-steps whose running
// balance never goes negative.  The ballot-word library `binary_words.ac`
// models exactly this: `successful_ballot_words(n, n)` lists the words with
// n positives and n negatives whose prefixes stay nonnegative, and the ballot
// reflection results of `ballot_reflection.ac` are the machinery that counts
// them.  The count of ALL ballot words with p positives and q negatives is
// already proved there as `ballot_words_length_binom(p, q)`:
//
//     ballot_words(p, q).length = (p + q).binom(p)
//
// which is the number of monotone lattice paths from (0, 0) to (p, q) that
// use p horizontal and q vertical steps, each ballot word corresponding to a
// path.  The Dyck-word count `successful_ballot_words(n, n).length = C_n`
// follows from the reflection bijection of `ballot_reflection.ac` together
// with the closed form `catalan_mul_suc` proved here; the bijection step is
// left for later work.
