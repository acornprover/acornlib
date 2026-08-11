/// The partition numbers.
///
/// This file deepens the small-value table of `combinatorics.partitions`,
/// which proves p(0) = 1, p(1) = 1, p(2) = 2, p(3) = 3 and p(4) = 5.  Here
/// the next two values are proved by the same explicit enumeration:
///
///     p(5) = 7   (5, 4+1, 3+2, 3+1+1, 2+2+1, 2+1+1+1, 1+1+1+1+1),
///     p(6) = 11  (6, 5+1, 4+2, 4+1+1, 3+3, 3+2+1, 3+1+1+1, 2+2+2,
///                  2+2+1+1, 2+1+1+1+1, 1+1+1+1+1+1).
///
/// Euler's odd-distinct identity is verified for n = 5: the partitions of 5
/// into distinct parts are 5, 4+1 and 3+2 (three of them), and the
/// partitions of 5 into odd parts are 5, 3+1+1 and 1+1+1+1+1 (three of
/// them).  Euler's pentagonal recurrence is verified for n = 5:
///
///     p(5) = p(4) + p(3) - p(0) = 5 + 3 - 1 = 7,
///
/// the case of the general recurrence p(n) = p(n-1) + p(n-2) - p(n-5) - ...,
/// which is recorded (together with the generating function identity) as a
/// statement for later work at the bottom of the file.  Finally, the growth
/// of `p` is verified through the new values: p(1) < ... < p(6).

from nat import Nat, add_comm, add_assoc, add_cancels_left, add_cancels_right,
    add_sub, sub_self, lte_trans, lte_suc_suc, lt_suc, lt_imp_lte_suc,
    lt_imp_lt_suc, lt_or_lte, lte_antisymm, lte_cancel_suc, lt_cancel_suc,
    add_to_zero, lte_add_left, lte_add_right, lt_add_left, lte_ref,
    lte_and_lt, lt_and_lte, lt_not_symm, lte_imp_not_lt, lt_trans, one_plus_one,
    add_one_right, add_one_left, add_zero_left, lt_not_ref, not_lt_zero,
    lt_suc_right, trichotomy, add_imp_sub, add_imp_sub_left, sub_zero,
    suc_sub_one, sub_lt
from list import List, sum, map, add_contains_left, add_contains_right,
    filter_contains_and, filter_contained_by_and, filter_equivalent_to_and,
    unique_implies_tail_unique, unique_length, singleton_unique,
    map_contains_of_contains, range_contains_of_lt,
    cons_unique_of_tail_unique_not_contains
from data.basic.logic import or_intro_left, or_intro_right
from data.list.list_cons_membership import cons_contains_eq, cons_contains_of_tail_contains,
    cons_contains_head, nil_not_contains
from data.list.list_unique_cons import unique_cons_head_not_in_tail
from data.basic.set import Set, contains_set_intro, finite_constraint,
    cardinality_always_exists, cardinality_is_well_defined
from combinatorics.partitions import is_partition, is_partition_sum,
    is_partition_all_positive, is_partition_non_increasing, is_partition_intro,
    all_positive, all_positive_head, all_positive_tail, non_increasing,
    non_increasing_from, non_increasing_second, non_increasing_tail, partition_set,
    partition_set_contains_iff, p, p_zero, p_one, p_two, p_three, p_four,
    partition_three_char, partition_four_char, p4_a, p4_b, p4_c, p4_d, p4_e,
    p4_sound, pair_is_unique,
    triple_is_unique, quad_is_unique, cons_neq_head, cons_cons_neq_head,
    bool_eq_of_iff, contains_le_sum, filter_all_keep, sum_ge_length_all_positive,
    zero_lte, lt_two_cases, lt_three_cases, lt_four_cases, ni_single, ni_21,
    ni_31, ni_22, ni_111, ni_211, ni_1111, ap_single, ap_21, ap_31, ap_22,
    ap_111, ap_211, ap_1111, partition_set_finite
from combinatorics.partition_identities import is_odd, is_odd_suc, is_odd_zero,
    is_odd_one, is_odd_two, is_odd_three, is_odd_four, all_odd,
    is_odd_part_partition, is_distinct_part_partition, odd_partition_set,
    distinct_partition_set, odd_partition_set_contains_iff,
    distinct_partition_set_contains_iff, all_odd_31, all_odd_1111, all_odd_4,
    all_odd_22, all_odd_211, pair_nat_unique, not_unique_22, not_unique_211,
    not_unique_1111, p_grows_one_two, p_grows_two_three, p_grows_three_four

numerals Nat

// ---------------------------------------------------------------------------
// Small case lemmas for the arithmetic below.
// ---------------------------------------------------------------------------

/// A number below five is zero, one, two, three or four.
theorem lt_five_cases(a: Nat) {
    a < Nat.5 implies (a = Nat.4 or a = Nat.3 or a = Nat.2 or a = Nat.1 or a = Nat.0)
} by {
    if a < Nat.5 {
        lt_suc_right(a, Nat.4)
        a < Nat.5 implies a = Nat.4 or a < Nat.4
        a = Nat.4 or a < Nat.4
        if not a = Nat.4 {
            a < Nat.4
            lt_four_cases(a)
            a = Nat.3 or a = Nat.2 or a = Nat.1 or a = Nat.0
            a = Nat.4 or a = Nat.3 or a = Nat.2 or a = Nat.1 or a = Nat.0
        }
        a = Nat.4 or a = Nat.3 or a = Nat.2 or a = Nat.1 or a = Nat.0
    }
}

/// A number below six is zero, one, two, three, four or five.
theorem lt_six_cases(a: Nat) {
    a < Nat.6 implies (a = Nat.5 or a = Nat.4 or a = Nat.3 or a = Nat.2 or a = Nat.1 or a = Nat.0)
} by {
    if a < Nat.6 {
        lt_suc_right(a, Nat.5)
        a < Nat.6 implies a = Nat.5 or a < Nat.5
        a = Nat.5 or a < Nat.5
        if not a = Nat.5 {
            a < Nat.5
            lt_five_cases(a)
            a = Nat.4 or a = Nat.3 or a = Nat.2 or a = Nat.1 or a = Nat.0
            a = Nat.5 or a = Nat.4 or a = Nat.3 or a = Nat.2 or a = Nat.1 or a = Nat.0
        }
        a = Nat.5 or a = Nat.4 or a = Nat.3 or a = Nat.2 or a = Nat.1 or a = Nat.0
    }
}
// ---------------------------------------------------------------------------
// Two, three, four and five positive ordered parts summing to five.
// ---------------------------------------------------------------------------

/// The only two positive ordered parts summing to five are 4, 1 and 3, 2.
theorem two_parts_five(a: Nat, b: Nat) {
    Nat.1 <= b and b <= a and a + b = Nat.5 implies (a = Nat.4 and b = Nat.1) or (a = Nat.3 and b = Nat.2)
} by {
    if Nat.1 <= b and b <= a and a + b = Nat.5 {
        Nat.1 <= b
        b <= a
        a + b = Nat.5
        lte_add_left(a, Nat.1, b)
        Nat.1 <= b implies a + Nat.1 <= a + b
        a + Nat.1 <= a + b
        a + Nat.1 <= Nat.5
        add_one_right(a)
        a + Nat.1 = a.suc
        a.suc <= Nat.5
        lt_or_lte(a, Nat.4)
        a < Nat.4 or Nat.4 <= a
        if a < Nat.4 {
            lt_four_cases(a)
            a = Nat.3 or a = Nat.2 or a = Nat.1 or a = Nat.0
            if a = Nat.3 {
                Nat.3 + b = Nat.5
                Nat.5 = Nat.3 + Nat.2
                add_cancels_left(Nat.3, b, Nat.2)
                b = Nat.2
                a = Nat.3 and b = Nat.2
                or_intro_right(a = Nat.4 and b = Nat.1, a = Nat.3 and b = Nat.2)
                a = Nat.3 and b = Nat.2 implies (a = Nat.4 and b = Nat.1) or (a = Nat.3 and b = Nat.2)
                (a = Nat.4 and b = Nat.1) or (a = Nat.3 and b = Nat.2)
            }
            if not a = Nat.3 {
                if a = Nat.2 {
                    Nat.2 + b = Nat.5
                    Nat.5 = Nat.2 + Nat.3
                    add_cancels_left(Nat.2, b, Nat.3)
                    b = Nat.3
                    b <= a
                    Nat.3 <= Nat.2
                    lte_imp_not_lt(Nat.3, Nat.2)
                    Nat.3 <= Nat.2 implies not Nat.2 < Nat.3
                    not Nat.2 < Nat.3
                    lt_suc(Nat.2)
                    Nat.2 < Nat.3
                    false
                }
                if not a = Nat.2 {
                    if a = Nat.1 {
                        Nat.1 + b = Nat.5
                        Nat.5 = Nat.1 + Nat.4
                        add_cancels_left(Nat.1, b, Nat.4)
                        b = Nat.4
                        b <= a
                        Nat.4 <= Nat.1
                        lte_imp_not_lt(Nat.4, Nat.1)
                        Nat.4 <= Nat.1 implies not Nat.1 < Nat.4
                        not Nat.1 < Nat.4
                        lt_suc(Nat.1)
                        Nat.1 < Nat.2
                        lt_suc(Nat.2)
                        Nat.2 < Nat.3
                        lt_suc(Nat.3)
                        Nat.3 < Nat.4
                        lt_trans(Nat.2, Nat.3, Nat.4)
                        Nat.2 < Nat.4
                        lt_trans(Nat.1, Nat.2, Nat.4)
                        Nat.1 < Nat.4
                        false
                    }
                    if not a = Nat.1 {
                        a = Nat.0
                        lte_trans(Nat.1, b, a)
                        Nat.1 <= b and b <= a implies Nat.1 <= a
                        Nat.1 <= a
                        Nat.1 <= Nat.0
                        false
                    }
                    false
                }
                false
            }
            (a = Nat.4 and b = Nat.1) or (a = Nat.3 and b = Nat.2)
        }
        if not a < Nat.4 {
            a.suc <= Nat.5
            lte_cancel_suc(a, Nat.4)
            a.suc <= Nat.4.suc implies a <= Nat.4
            a <= Nat.4
            lte_imp_not_lt(a, Nat.4)
            a <= Nat.4 implies not Nat.4 < a
            lt_or_lte(Nat.4, a)
            Nat.4 < a or a <= Nat.4
            if Nat.4 < a {
                not Nat.4 < a
                false
            }
            if not Nat.4 < a {
                a <= Nat.4
            }
            a <= Nat.4
            lte_antisymm(a, Nat.4)
            a = Nat.4
            Nat.4 + b = Nat.5
            Nat.5 = Nat.4 + Nat.1
            add_cancels_left(Nat.4, b, Nat.1)
            b = Nat.1
            a = Nat.4 and b = Nat.1
            or_intro_left(a = Nat.4 and b = Nat.1, a = Nat.3 and b = Nat.2)
            a = Nat.4 and b = Nat.1 implies (a = Nat.4 and b = Nat.1) or (a = Nat.3 and b = Nat.2)
            (a = Nat.4 and b = Nat.1) or (a = Nat.3 and b = Nat.2)
        }
        (a = Nat.4 and b = Nat.1) or (a = Nat.3 and b = Nat.2)
    }
}

/// The only three positive ordered parts summing to five are 3, 1, 1 and
/// 2, 2, 1.
theorem three_parts_five(a: Nat, b: Nat, c: Nat) {
    Nat.1 <= a and Nat.1 <= b and Nat.1 <= c and b <= a and c <= b and
        a + b + c = Nat.5 implies (a = Nat.3 and b = Nat.1 and c = Nat.1) or (a = Nat.2 and b = Nat.2 and c = Nat.1)
} by {
    if Nat.1 <= a and Nat.1 <= b and Nat.1 <= c and b <= a and c <= b and
        a + b + c = Nat.5 {
        Nat.1 <= a
        Nat.1 <= b
        Nat.1 <= c
        b <= a
        c <= b
        a + b + c = Nat.5
        add_assoc(a, b, c)
        a + b + c = a + (b + c)
        a + (b + c) = Nat.5
        lt_or_lte(a, Nat.3)
        a < Nat.3 or Nat.3 <= a
        if a < Nat.3 {
            lt_three_cases(a)
            a = Nat.2 or a = Nat.1 or a = Nat.0
            if a = Nat.2 {
                Nat.2 + (b + c) = Nat.5
                Nat.5 = Nat.2 + Nat.3
                add_cancels_left(Nat.2, b + c, Nat.3)
                b + c = Nat.3
                lt_or_lte(b, Nat.2)
                b < Nat.2 or Nat.2 <= b
                if b < Nat.2 {
                    lt_two_cases(b)
                    b = Nat.1 or b = Nat.0
                    if b = Nat.1 {
                        Nat.1 + c = Nat.3
                        Nat.3 = Nat.1 + Nat.2
                        add_cancels_left(Nat.1, c, Nat.2)
                        c = Nat.2
                        c <= b
                        Nat.2 <= Nat.1
                        lte_imp_not_lt(Nat.2, Nat.1)
                        Nat.2 <= Nat.1 implies not Nat.1 < Nat.2
                        not Nat.1 < Nat.2
                        lt_suc(Nat.1)
                        Nat.1 < Nat.2
                        false
                    }
                    if not b = Nat.1 {
                        b = Nat.0
                        Nat.1 <= b
                        Nat.1 <= Nat.0
                        false
                    }
                    false
                }
                if not b < Nat.2 {
                    b <= a
                    b <= Nat.2
                    lt_or_lte(b, Nat.2)
                    b < Nat.2 or Nat.2 <= b
                    if b < Nat.2 {
                        false
                    }
                    if not b < Nat.2 {
                        Nat.2 <= b
                    }
                    Nat.2 <= b
                    lte_antisymm(b, Nat.2)
                    b = Nat.2
                    Nat.2 + c = Nat.3
                    Nat.3 = Nat.2 + Nat.1
                    add_cancels_left(Nat.2, c, Nat.1)
                    c = Nat.1
                    a = Nat.2 and b = Nat.2 and c = Nat.1
                    or_intro_right(a = Nat.3 and b = Nat.1 and c = Nat.1, a = Nat.2 and b = Nat.2 and c = Nat.1)
                    a = Nat.2 and b = Nat.2 and c = Nat.1 implies (a = Nat.3 and b = Nat.1 and c = Nat.1) or (a = Nat.2 and b = Nat.2 and c = Nat.1)
                    (a = Nat.3 and b = Nat.1 and c = Nat.1) or (a = Nat.2 and b = Nat.2 and c = Nat.1)
                }
                (a = Nat.3 and b = Nat.1 and c = Nat.1) or (a = Nat.2 and b = Nat.2 and c = Nat.1)
            }
            if not a = Nat.2 {
                if a = Nat.1 {
                    Nat.1 + (b + c) = Nat.5
                    Nat.5 = Nat.1 + Nat.4
                    add_cancels_left(Nat.1, b + c, Nat.4)
                    b + c = Nat.4
                    lte_antisymm(b, Nat.1)
                    b <= a
                    Nat.1 <= b
                    b = Nat.1
                    Nat.1 + c = Nat.4
                    Nat.4 = Nat.1 + Nat.3
                    add_cancels_left(Nat.1, c, Nat.3)
                    c = Nat.3
                    c <= b
                    Nat.3 <= Nat.1
                    lte_imp_not_lt(Nat.3, Nat.1)
                    Nat.3 <= Nat.1 implies not Nat.1 < Nat.3
                    not Nat.1 < Nat.3
                    lt_suc(Nat.1)
                    Nat.1 < Nat.2
                    lt_suc(Nat.2)
                    Nat.2 < Nat.3
                    lt_trans(Nat.1, Nat.2, Nat.3)
                    Nat.1 < Nat.3
                    false
                }
                if not a = Nat.1 {
                    a = Nat.0
                    Nat.1 <= a
                    Nat.1 <= Nat.0
                    false
                }
                false
            }
            (a = Nat.3 and b = Nat.1 and c = Nat.1) or (a = Nat.2 and b = Nat.2 and c = Nat.1)
        }
        if not a < Nat.3 {
            lte_add_left(b, Nat.1, c)
            Nat.1 <= c implies b + Nat.1 <= b + c
            b + Nat.1 <= b + c
            add_one_right(b)
            b + Nat.1 = b.suc
            b.suc <= b + c
            lte_add_right(Nat.1, b, Nat.1)
            Nat.1 <= b implies Nat.1 + Nat.1 <= b + Nat.1
            Nat.1 + Nat.1 = Nat.2
            Nat.2 <= b + Nat.1
            add_one_right(b)
            b + Nat.1 = b.suc
            Nat.2 <= b.suc
            lte_trans(Nat.2, b.suc, b + c)
            Nat.2 <= b + c
            lte_add_left(a, Nat.2, b + c)
            Nat.2 <= b + c implies a + Nat.2 <= a + (b + c)
            a + Nat.2 <= a + (b + c)
            a + (b + c) = Nat.5
            a + Nat.2 <= Nat.5
            add_one_right(a)
            a + Nat.1 = a.suc
            add_one_right(a.suc)
            a.suc + Nat.1 = a.suc.suc
            a + Nat.2 = a.suc.suc
            a.suc.suc <= Nat.5
            lte_cancel_suc(a.suc, Nat.4)
            a.suc.suc <= Nat.4.suc implies a.suc <= Nat.4
            a.suc <= Nat.4
            lte_cancel_suc(a, Nat.3)
            a.suc <= Nat.3.suc implies a <= Nat.3
            a <= Nat.3
            lte_antisymm(a, Nat.3)
            a = Nat.3
            Nat.3 + (b + c) = Nat.5
            Nat.5 = Nat.3 + Nat.2
            add_cancels_left(Nat.3, b + c, Nat.2)
            b + c = Nat.2
            lt_or_lte(b, Nat.1)
            b < Nat.1 or Nat.1 <= b
            if b < Nat.1 {
                b = Nat.0
                Nat.1 <= b
                Nat.1 <= Nat.0
                false
            }
            if not b < Nat.1 {
                Nat.1 <= b
                lte_imp_not_lt(b, Nat.1)
                b <= Nat.1 implies not Nat.1 < b
                lte_add_left(b, Nat.1, c)
                Nat.1 <= c implies b + Nat.1 <= b + c
                b + Nat.1 <= b + c
                add_one_right(b)
                b + Nat.1 = b.suc
                b.suc <= b + c
                b.suc <= Nat.2
                lte_cancel_suc(b, Nat.1)
                b.suc <= Nat.1.suc implies b <= Nat.1
                b <= Nat.1
                lte_antisymm(b, Nat.1)
                b = Nat.1
                Nat.1 + c = Nat.2
                Nat.2 = Nat.1 + Nat.1
                add_cancels_left(Nat.1, c, Nat.1)
                c = Nat.1
                a = Nat.3 and b = Nat.1 and c = Nat.1
                or_intro_left(a = Nat.3 and b = Nat.1 and c = Nat.1, a = Nat.2 and b = Nat.2 and c = Nat.1)
                a = Nat.3 and b = Nat.1 and c = Nat.1 implies (a = Nat.3 and b = Nat.1 and c = Nat.1) or (a = Nat.2 and b = Nat.2 and c = Nat.1)
                (a = Nat.3 and b = Nat.1 and c = Nat.1) or (a = Nat.2 and b = Nat.2 and c = Nat.1)
            }
            (a = Nat.3 and b = Nat.1 and c = Nat.1) or (a = Nat.2 and b = Nat.2 and c = Nat.1)
        }
        (a = Nat.3 and b = Nat.1 and c = Nat.1) or (a = Nat.2 and b = Nat.2 and c = Nat.1)
    }
}
/// The only four positive ordered parts summing to five are 2, 1, 1, 1.
theorem four_parts_five(a: Nat, b: Nat, c: Nat, d: Nat) {
    Nat.1 <= a and Nat.1 <= b and Nat.1 <= c and Nat.1 <= d and b <= a and c <= b and d <= c and
        a + b + c + d = Nat.5 implies a = Nat.2 and b = Nat.1 and c = Nat.1 and d = Nat.1
} by {
    if Nat.1 <= a and Nat.1 <= b and Nat.1 <= c and Nat.1 <= d and b <= a and c <= b and d <= c and
        a + b + c + d = Nat.5 {
        Nat.1 <= a
        Nat.1 <= b
        Nat.1 <= c
        Nat.1 <= d
        b <= a
        c <= b
        d <= c
        a + b + c + d = Nat.5
        add_assoc(a, b, c + d)
        a + (b + (c + d)) = Nat.5
        lt_or_lte(a, Nat.1)
        a < Nat.1 or Nat.1 <= a
        if a < Nat.1 {
            a = Nat.0
            Nat.1 <= a
            Nat.1 <= Nat.0
            false
        }
        if not a < Nat.1 {
            Nat.1 <= a
            lte_add_left(c, Nat.1, d)
            Nat.1 <= d implies c + Nat.1 <= c + d
            c + Nat.1 <= c + d
            add_one_right(c)
            c + Nat.1 = c.suc
            c.suc <= c + d
            lte_add_right(Nat.1, c, Nat.1)
            Nat.1 <= c implies Nat.1 + Nat.1 <= c + Nat.1
            Nat.1 + Nat.1 = Nat.2
            Nat.2 <= c + Nat.1
            add_one_right(c)
            c + Nat.1 = c.suc
            Nat.2 <= c.suc
            lte_trans(Nat.2, c.suc, c + d)
            Nat.2 <= c + d
            lte_add_left(b, Nat.2, c + d)
            Nat.2 <= c + d implies b + Nat.2 <= b + (c + d)
            b + Nat.2 <= b + (c + d)
            lte_add_right(Nat.2, b, Nat.1)
            Nat.1 <= b implies Nat.1 + Nat.2 <= b + Nat.2
            Nat.1 + Nat.2 = Nat.3
            Nat.3 <= b + Nat.2
            lte_trans(Nat.3, b + Nat.2, b + (c + d))
            Nat.3 <= b + (c + d)
            lte_add_left(a, Nat.3, b + (c + d))
            Nat.3 <= b + (c + d) implies a + Nat.3 <= a + (b + (c + d))
            a + Nat.3 <= a + (b + (c + d))
            a + (b + (c + d)) = Nat.5
            a + Nat.3 <= Nat.5
            add_one_right(a)
            a + Nat.1 = a.suc
            add_one_right(a.suc)
            a.suc + Nat.1 = a.suc.suc
            a + Nat.2 = a.suc.suc
            a + Nat.3 = a.suc.suc.suc
            a.suc.suc.suc <= Nat.5
            lte_cancel_suc(a.suc.suc, Nat.4)
            a.suc.suc <= Nat.4
            lte_cancel_suc(a.suc, Nat.3)
            a.suc <= Nat.3
            lte_cancel_suc(a, Nat.2)
            a <= Nat.2
            lt_imp_lt_suc(a, Nat.2)
            a <= Nat.2 implies a < Nat.3
            a < Nat.3
            lt_three_cases(a)
            a = Nat.2 or a = Nat.1 or a = Nat.0
            if a = Nat.2 {
                Nat.2 + (b + (c + d)) = Nat.5
                add_cancels_left(Nat.2, b + (c + d), Nat.3)
                b + (c + d) = Nat.3
                lt_or_lte(b, Nat.2)
                b < Nat.2 or Nat.2 <= b
                if b < Nat.2 {
                    lt_two_cases(b)
                    b = Nat.1 or b = Nat.0
                    if b = Nat.1 {
                        Nat.1 + (c + d) = Nat.3
                        add_cancels_left(Nat.1, c + d, Nat.2)
                        c + d = Nat.2
                        lte_antisymm(c, Nat.1)
                        c <= b
                        Nat.1 <= c
                        c = Nat.1
                        Nat.1 + d = Nat.2
                        add_cancels_left(Nat.1, d, Nat.1)
                        d = Nat.1
                        a = Nat.2 and b = Nat.1 and c = Nat.1 and d = Nat.1
                    }
                    if not b = Nat.1 {
                        b = Nat.0
                        Nat.1 <= b
                        Nat.1 <= Nat.0
                        false
                    }
                    a = Nat.2 and b = Nat.1 and c = Nat.1 and d = Nat.1
                }
                if not b < Nat.2 {
                    b <= a
                    b <= Nat.2
                    lt_or_lte(b, Nat.2)
                    b < Nat.2 or Nat.2 <= b
                    if b < Nat.2 {
                        false
                    }
                    if not b < Nat.2 {
                        Nat.2 <= b
                    }
                    Nat.2 <= b
                    lte_antisymm(b, Nat.2)
                    b = Nat.2
                    Nat.2 + (c + d) = Nat.3
                    add_cancels_left(Nat.2, c + d, Nat.1)
                    c + d = Nat.1
                    lte_add_left(c, Nat.1, d)
                    Nat.1 <= d implies c + Nat.1 <= c + d
                    c + Nat.1 <= c + d
                    add_one_right(c)
                    c + Nat.1 = c.suc
                    c.suc <= c + d
                    lte_add_right(Nat.1, c, Nat.1)
                    Nat.1 <= c implies Nat.1 + Nat.1 <= c + Nat.1
                    Nat.1 + Nat.1 = Nat.2
                    Nat.2 <= c + Nat.1
                    add_one_right(c)
                    c + Nat.1 = c.suc
                    Nat.2 <= c.suc
                    lte_trans(Nat.2, c.suc, c + d)
                    Nat.2 <= c + d
                    c + d = Nat.1
                    Nat.2 <= Nat.1
                    lte_imp_not_lt(Nat.2, Nat.1)
                    Nat.2 <= Nat.1 implies not Nat.1 < Nat.2
                    not Nat.1 < Nat.2
                    lt_suc(Nat.1)
                    Nat.1 < Nat.2
                    false
                }
                a = Nat.2 and b = Nat.1 and c = Nat.1 and d = Nat.1
            }
            if not a = Nat.2 {
                if a = Nat.1 {
                    Nat.1 + (b + (c + d)) = Nat.5
                    add_cancels_left(Nat.1, b + (c + d), Nat.4)
                    b + (c + d) = Nat.4
                    lte_antisymm(b, Nat.1)
                    b <= a
                    Nat.1 <= b
                    b = Nat.1
                    Nat.1 + (c + d) = Nat.4
                    add_cancels_left(Nat.1, c + d, Nat.3)
                    c + d = Nat.3
                    lte_antisymm(c, Nat.1)
                    c <= b
                    Nat.1 <= c
                    c = Nat.1
                    Nat.1 + d = Nat.3
                    add_cancels_left(Nat.1, d, Nat.2)
                    d = Nat.2
                    d <= c
                    Nat.2 <= Nat.1
                    lte_imp_not_lt(Nat.2, Nat.1)
                    Nat.2 <= Nat.1 implies not Nat.1 < Nat.2
                    not Nat.1 < Nat.2
                    lt_suc(Nat.1)
                    Nat.1 < Nat.2
                    false
                }
                if not a = Nat.1 {
                    a = Nat.0
                    Nat.1 <= a
                    Nat.1 <= Nat.0
                    false
                }
                false
            }
            a = Nat.2 and b = Nat.1 and c = Nat.1 and d = Nat.1
        }
        a = Nat.2 and b = Nat.1 and c = Nat.1 and d = Nat.1
    }
}

/// The only five positive ordered parts summing to five are all ones.
theorem five_parts_five(a: Nat, b: Nat, c: Nat, d: Nat, e: Nat) {
    Nat.1 <= a and Nat.1 <= b and Nat.1 <= c and Nat.1 <= d and Nat.1 <= e and
        b <= a and c <= b and d <= c and e <= d and
        a + (b + (c + (d + e))) = Nat.5 implies a = Nat.1 and b = Nat.1 and c = Nat.1 and d = Nat.1 and e = Nat.1
} by {
    if Nat.1 <= a and Nat.1 <= b and Nat.1 <= c and Nat.1 <= d and Nat.1 <= e and
        b <= a and c <= b and d <= c and e <= d and
        a + (b + (c + (d + e))) = Nat.5 {
        Nat.1 <= a
        Nat.1 <= b
        Nat.1 <= c
        Nat.1 <= d
        Nat.1 <= e
        b <= a
        c <= b
        d <= c
        e <= d
        a + (b + (c + (d + e))) = Nat.5
        lte_add_left(d, Nat.1, e)
        Nat.1 <= e implies d + Nat.1 <= d + e
        d + Nat.1 <= d + e
        add_one_right(d)
        d + Nat.1 = d.suc
        d.suc <= d + e
        lte_add_right(Nat.1, d, Nat.1)
        Nat.1 <= d implies Nat.1 + Nat.1 <= d + Nat.1
        Nat.1 + Nat.1 = Nat.2
        Nat.2 <= d + Nat.1
        add_one_right(d)
        d + Nat.1 = d.suc
        Nat.2 <= d.suc
        lte_trans(Nat.2, d.suc, d + e)
        Nat.2 <= d + e
        lte_add_left(c, Nat.2, d + e)
        Nat.2 <= d + e implies c + Nat.2 <= c + (d + e)
        c + Nat.2 <= c + (d + e)
        lte_add_right(Nat.2, c, Nat.1)
        Nat.1 <= c implies Nat.1 + Nat.2 <= c + Nat.2
        Nat.1 + Nat.2 = Nat.3
        Nat.3 <= c + Nat.2
        lte_trans(Nat.3, c + Nat.2, c + (d + e))
        Nat.3 <= c + (d + e)
        lte_add_left(b, Nat.3, c + (d + e))
        Nat.3 <= c + (d + e) implies b + Nat.3 <= b + (c + (d + e))
        b + Nat.3 <= b + (c + (d + e))
        lte_add_right(Nat.3, b, Nat.1)
        Nat.1 <= b implies Nat.1 + Nat.3 <= b + Nat.3
        Nat.1 + Nat.3 = Nat.4
        Nat.4 <= b + Nat.3
        lte_trans(Nat.4, b + Nat.3, b + (c + (d + e)))
        Nat.4 <= b + (c + (d + e))
        lte_add_left(a, Nat.4, b + (c + (d + e)))
        Nat.4 <= b + (c + (d + e)) implies a + Nat.4 <= a + (b + (c + (d + e)))
        a + Nat.4 <= a + (b + (c + (d + e)))
        a + (b + (c + (d + e))) = Nat.5
        a + Nat.4 <= Nat.5
        add_one_right(a)
        a + Nat.1 = a.suc
        add_one_right(a.suc)
        a.suc + Nat.1 = a.suc.suc
        a + Nat.2 = a.suc.suc
        a + Nat.3 = a.suc.suc.suc
        a + Nat.4 = a.suc.suc.suc.suc
        a.suc.suc.suc.suc <= Nat.5
        lte_cancel_suc(a.suc.suc.suc, Nat.4)
        a.suc.suc.suc <= Nat.4
        lte_cancel_suc(a.suc.suc, Nat.3)
        a.suc.suc <= Nat.3
        lte_cancel_suc(a.suc, Nat.2)
        a.suc <= Nat.2
        lte_cancel_suc(a, Nat.1)
        a <= Nat.1
        lte_antisymm(a, Nat.1)
        a = Nat.1
        a + (b + (c + (d + e))) = Nat.5
        add_cancels_left(Nat.1, b + (c + (d + e)), Nat.4)
        b + (c + (d + e)) = Nat.4
        lte_antisymm(b, Nat.1)
        b <= a
        Nat.1 <= b
        b = Nat.1
        b + (c + (d + e)) = Nat.4
        add_cancels_left(Nat.1, c + (d + e), Nat.3)
        c + (d + e) = Nat.3
        lte_antisymm(c, Nat.1)
        c <= b
        Nat.1 <= c
        c = Nat.1
        c + (d + e) = Nat.3
        add_cancels_left(Nat.1, d + e, Nat.2)
        d + e = Nat.2
        lte_antisymm(d, Nat.1)
        d <= c
        Nat.1 <= d
        d = Nat.1
        d + e = Nat.2
        add_cancels_left(Nat.1, e, Nat.1)
        e = Nat.1
        a = Nat.1 and b = Nat.1 and c = Nat.1 and d = Nat.1 and e = Nat.1
    }
}

// ---------------------------------------------------------------------------
// The partitions of five, by explicit enumeration.
// ---------------------------------------------------------------------------

/// The parts of the partitions of five, as named constants.
let p5_a: List[Nat] = List.cons(Nat.5, List.nil[Nat])
let p5_b: List[Nat] = List.cons(Nat.4, List.cons(Nat.1, List.nil[Nat]))
let p5_c: List[Nat] = List.cons(Nat.3, List.cons(Nat.2, List.nil[Nat]))
let p5_d: List[Nat] = List.cons(Nat.3, List.cons(Nat.1, List.cons(Nat.1, List.nil[Nat])))
let p5_e: List[Nat] = List.cons(Nat.2, List.cons(Nat.2, List.cons(Nat.1, List.nil[Nat])))
let p5_f: List[Nat] = List.cons(Nat.2, List.cons(Nat.1, List.cons(Nat.1, List.cons(Nat.1, List.nil[Nat]))))
let p5_g: List[Nat] = List.cons(Nat.1, List.cons(Nat.1, List.cons(Nat.1, List.cons(Nat.1, List.cons(Nat.1, List.nil[Nat])))))
// ---------------------------------------------------------------------------
// Small lemmas about positive lists of small sums, and about the parts of
// non-increasing lists.
// ---------------------------------------------------------------------------

/// A positive list of sum zero is empty.
theorem sum_zero_nil(t: List[Nat]) {
    sum(t) = Nat.0 and all_positive(t) implies t = List.nil[Nat]
} by {
    if sum(t) = Nat.0 and all_positive(t) {
        match t {
            List.nil {
                t = List.nil[Nat]
            }
            List.cons(b, t2) {
                sum(List.cons(b, t2)) = b + sum(t2)
                sum(t) = Nat.0
                b + sum(t2) = Nat.0
                add_to_zero(b, sum(t2))
                b = Nat.0 and sum(t2) = Nat.0
                all_positive(List.cons(b, t2))
                all_positive_head(b, t2)
                Nat.0 < b
                b != Nat.0
                false
            }
        }
        t = List.nil[Nat]
    }
}

/// A positive list of sum one is the singleton 1.
theorem sum_one_singleton(t: List[Nat]) {
    sum(t) = Nat.1 and all_positive(t) implies t = List.cons(Nat.1, List.nil[Nat])
} by {
    if sum(t) = Nat.1 and all_positive(t) {
        match t {
            List.nil {
                sum(List.nil[Nat]) = Nat.0
                sum(t) = Nat.1
                Nat.0 = Nat.1
                false
            }
            List.cons(b, t2) {
                sum(List.cons(b, t2)) = b + sum(t2)
                sum(t) = Nat.1
                b + sum(t2) = Nat.1
                all_positive(List.cons(b, t2))
                all_positive_head(b, t2)
                Nat.0 < b
                lt_imp_lte_suc(Nat.0, b)
                Nat.1 <= b
                zero_lte(sum(t2))
                Nat.0 <= sum(t2)
                lte_add_left(b, Nat.0, sum(t2))
                Nat.0 <= sum(t2) implies b + Nat.0 <= b + sum(t2)
                b + Nat.0 <= b + sum(t2)
                b + Nat.0 = b
                b <= b + sum(t2)
                b + sum(t2) = Nat.1
                b <= Nat.1
                lte_antisymm(b, Nat.1)
                b = Nat.1
                Nat.1 + sum(t2) = Nat.1
                add_cancels_left(Nat.1, sum(t2), Nat.0)
                sum(t2) = Nat.0
                all_positive_tail(b, t2)
                all_positive(t2)
                sum_zero_nil(t2)
                sum(t2) = Nat.0 and all_positive(t2) implies t2 = List.nil[Nat]
                t2 = List.nil[Nat]
                t = List.cons(b, t2)
                t = List.cons(Nat.1, List.nil[Nat])
            }
        }
        t = List.cons(Nat.1, List.nil[Nat])
    }
}

/// A positive list of sum two is 2 or 1 + 1.
theorem sum_two_cases(t: List[Nat]) {
    sum(t) = Nat.2 and all_positive(t) implies (t = List.cons(Nat.2, List.nil[Nat]) or t = List.cons(Nat.1, List.cons(Nat.1, List.nil[Nat])))
} by {
    if sum(t) = Nat.2 and all_positive(t) {
        match t {
            List.nil {
                sum(List.nil[Nat]) = Nat.0
                sum(t) = Nat.2
                Nat.0 = Nat.2
                false
            }
            List.cons(b, t2) {
                sum(List.cons(b, t2)) = b + sum(t2)
                sum(t) = Nat.2
                b + sum(t2) = Nat.2
                all_positive(List.cons(b, t2))
                all_positive_head(b, t2)
                Nat.0 < b
                lt_imp_lte_suc(Nat.0, b)
                Nat.1 <= b
                zero_lte(sum(t2))
                Nat.0 <= sum(t2)
                lte_add_left(b, Nat.0, sum(t2))
                Nat.0 <= sum(t2) implies b + Nat.0 <= b + sum(t2)
                b + Nat.0 <= b + sum(t2)
                b + Nat.0 = b
                b <= b + sum(t2)
                b + sum(t2) = Nat.2
                b <= Nat.2
                lt_imp_lt_suc(b, Nat.2)
                b <= Nat.2 implies b < Nat.3
                b < Nat.3
                lt_three_cases(b)
                b = Nat.2 or b = Nat.1 or b = Nat.0
                if b = Nat.2 {
                    Nat.2 + sum(t2) = Nat.2
                    add_cancels_left(Nat.2, sum(t2), Nat.0)
                    sum(t2) = Nat.0
                    all_positive_tail(b, t2)
                    all_positive(t2)
                    sum_zero_nil(t2)
                    sum(t2) = Nat.0 and all_positive(t2) implies t2 = List.nil[Nat]
                    t2 = List.nil[Nat]
                    t = List.cons(b, t2)
                    t = List.cons(Nat.2, List.nil[Nat])
                    or_intro_left(t = List.cons(Nat.2, List.nil[Nat]), t = List.cons(Nat.1, List.cons(Nat.1, List.nil[Nat])))
                    t = List.cons(Nat.2, List.nil[Nat]) implies (t = List.cons(Nat.2, List.nil[Nat]) or t = List.cons(Nat.1, List.cons(Nat.1, List.nil[Nat])))
                    t = List.cons(Nat.2, List.nil[Nat]) or t = List.cons(Nat.1, List.cons(Nat.1, List.nil[Nat]))
                }
                if not b = Nat.2 {
                    if b = Nat.1 {
                        Nat.1 + sum(t2) = Nat.2
                        add_cancels_left(Nat.1, sum(t2), Nat.1)
                        sum(t2) = Nat.1
                        all_positive_tail(b, t2)
                        all_positive(t2)
                        sum_one_singleton(t2)
                        sum(t2) = Nat.1 and all_positive(t2) implies t2 = List.cons(Nat.1, List.nil[Nat])
                        t2 = List.cons(Nat.1, List.nil[Nat])
                        t = List.cons(b, t2)
                        t = List.cons(Nat.1, List.cons(Nat.1, List.nil[Nat]))
                        or_intro_right(t = List.cons(Nat.2, List.nil[Nat]), t = List.cons(Nat.1, List.cons(Nat.1, List.nil[Nat])))
                        t = List.cons(Nat.1, List.cons(Nat.1, List.nil[Nat])) implies (t = List.cons(Nat.2, List.nil[Nat]) or t = List.cons(Nat.1, List.cons(Nat.1, List.nil[Nat])))
                        t = List.cons(Nat.2, List.nil[Nat]) or t = List.cons(Nat.1, List.cons(Nat.1, List.nil[Nat]))
                    }
                    if not b = Nat.1 {
                        b = Nat.0
                        Nat.1 <= b
                        Nat.1 <= Nat.0
                        false
                    }
                    t = List.cons(Nat.2, List.nil[Nat]) or t = List.cons(Nat.1, List.cons(Nat.1, List.nil[Nat]))
                }
                t = List.cons(Nat.2, List.nil[Nat]) or t = List.cons(Nat.1, List.cons(Nat.1, List.nil[Nat]))
            }
        }
        t = List.cons(Nat.2, List.nil[Nat]) or t = List.cons(Nat.1, List.cons(Nat.1, List.nil[Nat]))
    }
}

/// The tail of a non-increasing list is non-increasing.
theorem ni_tail_general(a: Nat, t: List[Nat]) {
    non_increasing(List.cons(a, t)) implies non_increasing(t)
} by {
    define q(xs: List[Nat], head: Nat) -> Bool {
        non_increasing_from(head, xs) implies non_increasing(xs)
    }
    define r(xs: List[Nat]) -> Bool {
        forall(hd: Nat) { q(xs, hd) }
    }
    forall(hd: Nat) {
        if non_increasing_from(hd, List.nil[Nat]) {
            non_increasing_from(hd, List.nil[Nat]) = true
            non_increasing(List.nil[Nat]) = true
        }
        q(List.nil[Nat], hd)
    }
    forall(hd: Nat) { q(List.nil[Nat], hd) }
    r(List.nil[Nat]) = forall(hd: Nat) { q(List.nil[Nat], hd) }
    r(List.nil[Nat])
    forall(h: Nat, t2: List[Nat]) {
        if r(t2) {
            r(t2) = forall(hd2: Nat) { q(t2, hd2) }
            forall(hd2: Nat) { q(t2, hd2) }
            forall(hd: Nat) {
                if non_increasing_from(hd, List.cons(h, t2)) {
                    non_increasing_from(hd, List.cons(h, t2)) =
                        (h <= hd and non_increasing_from(h, t2))
                    h <= hd
                    non_increasing_from(h, t2)
                    q(t2, h)
                    non_increasing_from(h, t2) implies non_increasing(t2)
                    non_increasing(t2)
                    non_increasing(List.cons(h, t2)) = non_increasing_from(h, t2)
                    non_increasing(List.cons(h, t2))
                }
                q(List.cons(h, t2), hd)
            }
            forall(hd: Nat) { q(List.cons(h, t2), hd) }
            r(List.cons(h, t2)) = forall(hd: Nat) { q(List.cons(h, t2), hd) }
            r(List.cons(h, t2))
        }
    }
    r(List.nil[Nat]) and forall(h: Nat, t2: List[Nat]) { r(t2) implies r(List.cons(h, t2)) }
    List.induction(function(xs: List[Nat]) { r(xs) })
    forall(xs: List[Nat]) { r(xs) }
    r(t)
    r(t) = forall(hd: Nat) { q(t, hd) }
    forall(hd: Nat) { q(t, hd) }
    q(t, a)
    non_increasing_from(a, t) implies non_increasing(t)
    non_increasing(List.cons(a, t)) = non_increasing_from(a, t)
    if non_increasing(List.cons(a, t)) {
        non_increasing_from(a, t)
        non_increasing(t)
    }
    non_increasing(List.cons(a, t)) implies non_increasing(t)
}

/// Every element of the tail of a non-increasing list is at most the head.
theorem parts_le_head(a: Nat, t: List[Nat], x: Nat) {
    non_increasing(List.cons(a, t)) and t.contains(x) implies x <= a
} by {
    define q(xs: List[Nat], head: Nat, y: Nat) -> Bool {
        non_increasing_from(head, xs) implies (xs.contains(y) implies y <= head)
    }
    define r(xs: List[Nat]) -> Bool {
        forall(hd: Nat) { q(xs, hd, x) }
    }
    forall(hd: Nat) {
        if List.nil[Nat].contains(x) {
            false
        }
        if non_increasing_from(hd, List.nil[Nat]) {
            q(List.nil[Nat], hd, x)
        }
        q(List.nil[Nat], hd, x)
    }
    forall(hd: Nat) { q(List.nil[Nat], hd, x) }
    r(List.nil[Nat]) = forall(hd: Nat) { q(List.nil[Nat], hd, x) }
    r(List.nil[Nat])
    forall(h: Nat, t2: List[Nat]) {
        if r(t2) {
            r(t2) = forall(hd2: Nat) { q(t2, hd2, x) }
            forall(hd2: Nat) { q(t2, hd2, x) }
            forall(hd: Nat) {
                if non_increasing_from(hd, List.cons(h, t2)) {
                    if List.cons(h, t2).contains(x) {
                        if h = x {
                            non_increasing_from(hd, List.cons(h, t2)) =
                                (h <= hd and non_increasing_from(h, t2))
                            h <= hd
                            x <= hd
                        }
                        if not h = x {
                            List.cons(h, t2).contains(x) implies t2.contains(x)
                            t2.contains(x)
                            non_increasing_from(h, t2)
                            q(t2, h, x)
                            non_increasing_from(h, t2) implies (t2.contains(x) implies x <= h)
                            t2.contains(x) implies x <= h
                            x <= h
                            non_increasing_from(hd, List.cons(h, t2)) =
                                (h <= hd and non_increasing_from(h, t2))
                            h <= hd
                            lte_trans(x, h, hd)
                            x <= hd
                        }
                        x <= hd
                    }
                    List.cons(h, t2).contains(x) implies x <= hd
                    q(List.cons(h, t2), hd, x)
                }
                q(List.cons(h, t2), hd, x)
            }
            forall(hd: Nat) { q(List.cons(h, t2), hd, x) }
            r(List.cons(h, t2)) = forall(hd: Nat) { q(List.cons(h, t2), hd, x) }
            r(List.cons(h, t2))
        }
    }
    r(List.nil[Nat]) and forall(h: Nat, t2: List[Nat]) { r(t2) implies r(List.cons(h, t2)) }
    List.induction(function(xs: List[Nat]) { r(xs) })
    forall(xs: List[Nat]) { r(xs) }
    r(t)
    r(t) = forall(hd: Nat) { q(t, hd, x) }
    forall(hd: Nat) { q(t, hd, x) }
    q(t, a, x)
    non_increasing_from(a, t) implies (t.contains(x) implies x <= a)
    non_increasing(List.cons(a, t)) = non_increasing_from(a, t)
    if non_increasing(List.cons(a, t)) and t.contains(x) {
        non_increasing_from(a, t)
        t.contains(x) implies x <= a
        x <= a
    }
    non_increasing(List.cons(a, t)) and t.contains(x) implies x <= a
}

/// The explicit list of the partitions of five.
let partitions_five: List[List[Nat]] =
    List.cons(p5_a,
        List.cons(p5_b,
            List.cons(p5_c,
                List.cons(p5_d,
                    List.cons(p5_e,
                        List.cons(p5_f,
                            List.cons(p5_g, List.nil[List[Nat]])))))))

/// Every partition of five occurs in the explicit list.
theorem partitions_five_contains(l: List[Nat]) {
    is_partition(l, Nat.5) implies partitions_five.contains(l)
} by {
    if is_partition(l, Nat.5) {
        is_partition_sum(l, Nat.5)
        sum(l) = Nat.5
        match l {
            List.nil {
                sum(List.nil[Nat]) = Nat.0
                Nat.0 = Nat.5
                false
            }
            List.cons(a, t) {
                sum(List.cons(a, t)) = a + sum(t)
                a + sum(t) = Nat.5
                is_partition_all_positive(l, Nat.5)
                all_positive(List.cons(a, t))
                all_positive_head(a, t)
                Nat.0 < a
                lt_imp_lte_suc(Nat.0, a)
                Nat.1 <= a
                all_positive_tail(a, t)
                all_positive(t)
                is_partition_non_increasing(l, Nat.5)
                non_increasing(List.cons(a, t))
                zero_lte(sum(t))
                Nat.0 <= sum(t)
                lte_add_left(a, Nat.0, sum(t))
                Nat.0 <= sum(t) implies a + Nat.0 <= a + sum(t)
                a + Nat.0 <= a + sum(t)
                a + Nat.0 = a
                a <= a + sum(t)
                a + sum(t) = Nat.5
                a <= Nat.5
                if a = Nat.5 {
                    Nat.5 + sum(t) = Nat.5
                    add_cancels_left(Nat.5, sum(t), Nat.0)
                    sum(t) = Nat.0
                    sum_zero_nil(t)
                    sum(t) = Nat.0 and all_positive(t) implies t = List.nil[Nat]
                    all_positive(t)
                    t = List.nil[Nat]
                    l = List.cons(a, t)
                    l = List.cons(Nat.5, List.nil[Nat])
                    p5_a = List.cons(Nat.5, List.nil[Nat])
                    l = p5_a
                    partitions_five = List.cons(p5_a, List.cons(p5_b, List.cons(p5_c, List.cons(p5_d, List.cons(p5_e, List.cons(p5_f, List.cons(p5_g, List.nil[List[Nat]])))))))
                    cons_contains_head(p5_a, List.cons(p5_b, List.cons(p5_c, List.cons(p5_d, List.cons(p5_e, List.cons(p5_f, List.cons(p5_g, List.nil[List[Nat]])))))))
                    List.cons(p5_a, List.cons(p5_b, List.cons(p5_c, List.cons(p5_d, List.cons(p5_e, List.cons(p5_f, List.cons(p5_g, List.nil[List[Nat]]))))))).contains(p5_a)
                    partitions_five.contains(p5_a)
                    partitions_five.contains(l)
                }
                if not a = Nat.5 {
                    if a = Nat.4 {
                        Nat.4 + sum(t) = Nat.5
                        add_cancels_left(Nat.4, sum(t), Nat.1)
                        sum(t) = Nat.1
                        sum_one_singleton(t)
                        sum(t) = Nat.1 and all_positive(t) implies t = List.cons(Nat.1, List.nil[Nat])
                        all_positive(t)
                        t = List.cons(Nat.1, List.nil[Nat])
                        l = List.cons(a, t)
                        l = List.cons(Nat.4, List.cons(Nat.1, List.nil[Nat]))
                        p5_b = List.cons(Nat.4, List.cons(Nat.1, List.nil[Nat]))
                        l = p5_b
                      partitions_five = List.cons(p5_a, List.cons(p5_b, List.cons(p5_c, List.cons(p5_d, List.cons(p5_e, List.cons(p5_f, List.cons(p5_g, List.nil[List[Nat]])))))))
                      cons_contains_head(p5_b, List.cons(p5_c, List.cons(p5_d, List.cons(p5_e, List.cons(p5_f, List.cons(p5_g, List.nil[List[Nat]]))))))
                      List.cons(p5_b, List.cons(p5_c, List.cons(p5_d, List.cons(p5_e, List.cons(p5_f, List.cons(p5_g, List.nil[List[Nat]])))))).contains(p5_b)
                      cons_contains_of_tail_contains(p5_a, List.cons(p5_b, List.cons(p5_c, List.cons(p5_d, List.cons(p5_e, List.cons(p5_f, List.cons(p5_g, List.nil[List[Nat]])))))), p5_b)
                      List.cons(p5_b, List.cons(p5_c, List.cons(p5_d, List.cons(p5_e, List.cons(p5_f, List.cons(p5_g, List.nil[List[Nat]])))))).contains(p5_b) implies List.cons(p5_a, List.cons(p5_b, List.cons(p5_c, List.cons(p5_d, List.cons(p5_e, List.cons(p5_f, List.cons(p5_g, List.nil[List[Nat]]))))))).contains(p5_b)
                      List.cons(p5_a, List.cons(p5_b, List.cons(p5_c, List.cons(p5_d, List.cons(p5_e, List.cons(p5_f, List.cons(p5_g, List.nil[List[Nat]]))))))).contains(p5_b)
                      partitions_five.contains(p5_b)
                      partitions_five.contains(l)
                    }
                    if not a = Nat.4 {
                        if a = Nat.3 {
                            Nat.3 + sum(t) = Nat.5
                            add_cancels_left(Nat.3, sum(t), Nat.2)
                            sum(t) = Nat.2
                            sum_two_cases(t)
                            sum(t) = Nat.2 and all_positive(t) implies (t = List.cons(Nat.2, List.nil[Nat]) or t = List.cons(Nat.1, List.cons(Nat.1, List.nil[Nat])))
                            all_positive(t)
                            t = List.cons(Nat.2, List.nil[Nat]) or t = List.cons(Nat.1, List.cons(Nat.1, List.nil[Nat]))
                            if t = List.cons(Nat.2, List.nil[Nat]) {
                                l = List.cons(a, t)
                                l = List.cons(Nat.3, List.cons(Nat.2, List.nil[Nat]))
                                p5_c = List.cons(Nat.3, List.cons(Nat.2, List.nil[Nat]))
                                l = p5_c
                            partitions_five = List.cons(p5_a, List.cons(p5_b, List.cons(p5_c, List.cons(p5_d, List.cons(p5_e, List.cons(p5_f, List.cons(p5_g, List.nil[List[Nat]])))))))
                            cons_contains_head(p5_c, List.cons(p5_d, List.cons(p5_e, List.cons(p5_f, List.cons(p5_g, List.nil[List[Nat]])))))
                            List.cons(p5_c, List.cons(p5_d, List.cons(p5_e, List.cons(p5_f, List.cons(p5_g, List.nil[List[Nat]]))))).contains(p5_c)
                            cons_contains_of_tail_contains(p5_b, List.cons(p5_c, List.cons(p5_d, List.cons(p5_e, List.cons(p5_f, List.cons(p5_g, List.nil[List[Nat]]))))), p5_c)
                            List.cons(p5_c, List.cons(p5_d, List.cons(p5_e, List.cons(p5_f, List.cons(p5_g, List.nil[List[Nat]]))))).contains(p5_c) implies List.cons(p5_b, List.cons(p5_c, List.cons(p5_d, List.cons(p5_e, List.cons(p5_f, List.cons(p5_g, List.nil[List[Nat]])))))).contains(p5_c)
                            List.cons(p5_b, List.cons(p5_c, List.cons(p5_d, List.cons(p5_e, List.cons(p5_f, List.cons(p5_g, List.nil[List[Nat]])))))).contains(p5_c)
                            cons_contains_of_tail_contains(p5_a, List.cons(p5_b, List.cons(p5_c, List.cons(p5_d, List.cons(p5_e, List.cons(p5_f, List.cons(p5_g, List.nil[List[Nat]])))))), p5_c)
                            List.cons(p5_b, List.cons(p5_c, List.cons(p5_d, List.cons(p5_e, List.cons(p5_f, List.cons(p5_g, List.nil[List[Nat]])))))).contains(p5_c) implies List.cons(p5_a, List.cons(p5_b, List.cons(p5_c, List.cons(p5_d, List.cons(p5_e, List.cons(p5_f, List.cons(p5_g, List.nil[List[Nat]]))))))).contains(p5_c)
                            List.cons(p5_a, List.cons(p5_b, List.cons(p5_c, List.cons(p5_d, List.cons(p5_e, List.cons(p5_f, List.cons(p5_g, List.nil[List[Nat]]))))))).contains(p5_c)
                            partitions_five.contains(p5_c)
                            partitions_five.contains(l)
                            }
                            if not t = List.cons(Nat.2, List.nil[Nat]) {
                                t = List.cons(Nat.1, List.cons(Nat.1, List.nil[Nat]))
                                l = List.cons(a, t)
                                l = List.cons(Nat.3, List.cons(Nat.1, List.cons(Nat.1, List.nil[Nat])))
                                p5_d = List.cons(Nat.3, List.cons(Nat.1, List.cons(Nat.1, List.nil[Nat])))
                                l = p5_d
                            partitions_five = List.cons(p5_a, List.cons(p5_b, List.cons(p5_c, List.cons(p5_d, List.cons(p5_e, List.cons(p5_f, List.cons(p5_g, List.nil[List[Nat]])))))))
                            cons_contains_head(p5_d, List.cons(p5_e, List.cons(p5_f, List.cons(p5_g, List.nil[List[Nat]]))))
                            List.cons(p5_d, List.cons(p5_e, List.cons(p5_f, List.cons(p5_g, List.nil[List[Nat]])))).contains(p5_d)
                            cons_contains_of_tail_contains(p5_c, List.cons(p5_d, List.cons(p5_e, List.cons(p5_f, List.cons(p5_g, List.nil[List[Nat]])))), p5_d)
                            List.cons(p5_d, List.cons(p5_e, List.cons(p5_f, List.cons(p5_g, List.nil[List[Nat]])))).contains(p5_d) implies List.cons(p5_c, List.cons(p5_d, List.cons(p5_e, List.cons(p5_f, List.cons(p5_g, List.nil[List[Nat]]))))).contains(p5_d)
                            List.cons(p5_c, List.cons(p5_d, List.cons(p5_e, List.cons(p5_f, List.cons(p5_g, List.nil[List[Nat]]))))).contains(p5_d)
                            cons_contains_of_tail_contains(p5_b, List.cons(p5_c, List.cons(p5_d, List.cons(p5_e, List.cons(p5_f, List.cons(p5_g, List.nil[List[Nat]]))))), p5_d)
                            List.cons(p5_c, List.cons(p5_d, List.cons(p5_e, List.cons(p5_f, List.cons(p5_g, List.nil[List[Nat]]))))).contains(p5_d) implies List.cons(p5_b, List.cons(p5_c, List.cons(p5_d, List.cons(p5_e, List.cons(p5_f, List.cons(p5_g, List.nil[List[Nat]])))))).contains(p5_d)
                            List.cons(p5_b, List.cons(p5_c, List.cons(p5_d, List.cons(p5_e, List.cons(p5_f, List.cons(p5_g, List.nil[List[Nat]])))))).contains(p5_d)
                            cons_contains_of_tail_contains(p5_a, List.cons(p5_b, List.cons(p5_c, List.cons(p5_d, List.cons(p5_e, List.cons(p5_f, List.cons(p5_g, List.nil[List[Nat]])))))), p5_d)
                            List.cons(p5_b, List.cons(p5_c, List.cons(p5_d, List.cons(p5_e, List.cons(p5_f, List.cons(p5_g, List.nil[List[Nat]])))))).contains(p5_d) implies List.cons(p5_a, List.cons(p5_b, List.cons(p5_c, List.cons(p5_d, List.cons(p5_e, List.cons(p5_f, List.cons(p5_g, List.nil[List[Nat]]))))))).contains(p5_d)
                            List.cons(p5_a, List.cons(p5_b, List.cons(p5_c, List.cons(p5_d, List.cons(p5_e, List.cons(p5_f, List.cons(p5_g, List.nil[List[Nat]]))))))).contains(p5_d)
                            partitions_five.contains(p5_d)
                            partitions_five.contains(l)
                            }
                            partitions_five.contains(l)
                        }
                        if not a = Nat.3 {
                            if a = Nat.2 {
                                Nat.2 + sum(t) = Nat.5
                                add_cancels_left(Nat.2, sum(t), Nat.3)
                                sum(t) = Nat.3
                                ni_tail_general(a, t)
                                non_increasing(List.cons(a, t)) implies non_increasing(t)
                                non_increasing(List.cons(a, t))
                                non_increasing(t)
                                is_partition_intro(t, Nat.3)
                                sum(t) = Nat.3 and non_increasing(t) and all_positive(t)
                                is_partition(t, Nat.3)
                                partition_three_char(t)
                                is_partition(t, Nat.3) = (t = List.cons(Nat.3, List.nil[Nat]) or t = List.cons(Nat.2, List.cons(Nat.1, List.nil[Nat])) or t = List.cons(Nat.1, List.cons(Nat.1, List.cons(Nat.1, List.nil[Nat]))))
                                t = List.cons(Nat.3, List.nil[Nat]) or t = List.cons(Nat.2, List.cons(Nat.1, List.nil[Nat])) or t = List.cons(Nat.1, List.cons(Nat.1, List.cons(Nat.1, List.nil[Nat])))
                                if t = List.cons(Nat.3, List.nil[Nat]) {
                                    cons_contains_head(Nat.3, List.nil[Nat])
                                    List.cons(Nat.3, List.nil[Nat]).contains(Nat.3)
                                    t.contains(Nat.3)
                                    parts_le_head(Nat.2, t, Nat.3)
                                    non_increasing(List.cons(Nat.2, t)) and t.contains(Nat.3) implies Nat.3 <= Nat.2
                                    non_increasing(List.cons(Nat.2, t))
                                    Nat.3 <= Nat.2
                                    lte_imp_not_lt(Nat.3, Nat.2)
                                    Nat.3 <= Nat.2 implies not Nat.2 < Nat.3
                                    not Nat.2 < Nat.3
                                    lt_suc(Nat.2)
                                    Nat.2 < Nat.3
                                    false
                                }
                                if not t = List.cons(Nat.3, List.nil[Nat]) {
                                    t = List.cons(Nat.2, List.cons(Nat.1, List.nil[Nat])) or t = List.cons(Nat.1, List.cons(Nat.1, List.cons(Nat.1, List.nil[Nat])))
                                    if t = List.cons(Nat.2, List.cons(Nat.1, List.nil[Nat])) {
                                        l = List.cons(a, t)
                                        l = List.cons(Nat.2, List.cons(Nat.2, List.cons(Nat.1, List.nil[Nat])))
                                        p5_e = List.cons(Nat.2, List.cons(Nat.2, List.cons(Nat.1, List.nil[Nat])))
                                        l = p5_e
                                  partitions_five = List.cons(p5_a, List.cons(p5_b, List.cons(p5_c, List.cons(p5_d, List.cons(p5_e, List.cons(p5_f, List.cons(p5_g, List.nil[List[Nat]])))))))
                                  cons_contains_head(p5_e, List.cons(p5_f, List.cons(p5_g, List.nil[List[Nat]])))
                                  List.cons(p5_e, List.cons(p5_f, List.cons(p5_g, List.nil[List[Nat]]))).contains(p5_e)
                                  cons_contains_of_tail_contains(p5_d, List.cons(p5_e, List.cons(p5_f, List.cons(p5_g, List.nil[List[Nat]]))), p5_e)
                                  List.cons(p5_e, List.cons(p5_f, List.cons(p5_g, List.nil[List[Nat]]))).contains(p5_e) implies List.cons(p5_d, List.cons(p5_e, List.cons(p5_f, List.cons(p5_g, List.nil[List[Nat]])))).contains(p5_e)
                                  List.cons(p5_d, List.cons(p5_e, List.cons(p5_f, List.cons(p5_g, List.nil[List[Nat]])))).contains(p5_e)
                                  cons_contains_of_tail_contains(p5_c, List.cons(p5_d, List.cons(p5_e, List.cons(p5_f, List.cons(p5_g, List.nil[List[Nat]])))), p5_e)
                                  List.cons(p5_d, List.cons(p5_e, List.cons(p5_f, List.cons(p5_g, List.nil[List[Nat]])))).contains(p5_e) implies List.cons(p5_c, List.cons(p5_d, List.cons(p5_e, List.cons(p5_f, List.cons(p5_g, List.nil[List[Nat]]))))).contains(p5_e)
                                  List.cons(p5_c, List.cons(p5_d, List.cons(p5_e, List.cons(p5_f, List.cons(p5_g, List.nil[List[Nat]]))))).contains(p5_e)
                                  cons_contains_of_tail_contains(p5_b, List.cons(p5_c, List.cons(p5_d, List.cons(p5_e, List.cons(p5_f, List.cons(p5_g, List.nil[List[Nat]]))))), p5_e)
                                  List.cons(p5_c, List.cons(p5_d, List.cons(p5_e, List.cons(p5_f, List.cons(p5_g, List.nil[List[Nat]]))))).contains(p5_e) implies List.cons(p5_b, List.cons(p5_c, List.cons(p5_d, List.cons(p5_e, List.cons(p5_f, List.cons(p5_g, List.nil[List[Nat]])))))).contains(p5_e)
                                  List.cons(p5_b, List.cons(p5_c, List.cons(p5_d, List.cons(p5_e, List.cons(p5_f, List.cons(p5_g, List.nil[List[Nat]])))))).contains(p5_e)
                                  cons_contains_of_tail_contains(p5_a, List.cons(p5_b, List.cons(p5_c, List.cons(p5_d, List.cons(p5_e, List.cons(p5_f, List.cons(p5_g, List.nil[List[Nat]])))))), p5_e)
                                  List.cons(p5_b, List.cons(p5_c, List.cons(p5_d, List.cons(p5_e, List.cons(p5_f, List.cons(p5_g, List.nil[List[Nat]])))))).contains(p5_e) implies List.cons(p5_a, List.cons(p5_b, List.cons(p5_c, List.cons(p5_d, List.cons(p5_e, List.cons(p5_f, List.cons(p5_g, List.nil[List[Nat]]))))))).contains(p5_e)
                                  List.cons(p5_a, List.cons(p5_b, List.cons(p5_c, List.cons(p5_d, List.cons(p5_e, List.cons(p5_f, List.cons(p5_g, List.nil[List[Nat]]))))))).contains(p5_e)
                                  partitions_five.contains(p5_e)
                                  partitions_five.contains(l)
                                    }
                                    if not t = List.cons(Nat.2, List.cons(Nat.1, List.nil[Nat])) {
                                        t = List.cons(Nat.1, List.cons(Nat.1, List.cons(Nat.1, List.nil[Nat])))
                                        l = List.cons(a, t)
                                        l = List.cons(Nat.2, List.cons(Nat.1, List.cons(Nat.1, List.cons(Nat.1, List.nil[Nat]))))
                                        p5_f = List.cons(Nat.2, List.cons(Nat.1, List.cons(Nat.1, List.cons(Nat.1, List.nil[Nat]))))
                                        l = p5_f
                                  partitions_five = List.cons(p5_a, List.cons(p5_b, List.cons(p5_c, List.cons(p5_d, List.cons(p5_e, List.cons(p5_f, List.cons(p5_g, List.nil[List[Nat]])))))))
                                  cons_contains_head(p5_f, List.cons(p5_g, List.nil[List[Nat]]))
                                  List.cons(p5_f, List.cons(p5_g, List.nil[List[Nat]])).contains(p5_f)
                                  cons_contains_of_tail_contains(p5_e, List.cons(p5_f, List.cons(p5_g, List.nil[List[Nat]])), p5_f)
                                  List.cons(p5_f, List.cons(p5_g, List.nil[List[Nat]])).contains(p5_f) implies List.cons(p5_e, List.cons(p5_f, List.cons(p5_g, List.nil[List[Nat]]))).contains(p5_f)
                                  List.cons(p5_e, List.cons(p5_f, List.cons(p5_g, List.nil[List[Nat]]))).contains(p5_f)
                                  cons_contains_of_tail_contains(p5_d, List.cons(p5_e, List.cons(p5_f, List.cons(p5_g, List.nil[List[Nat]]))), p5_f)
                                  List.cons(p5_e, List.cons(p5_f, List.cons(p5_g, List.nil[List[Nat]]))).contains(p5_f) implies List.cons(p5_d, List.cons(p5_e, List.cons(p5_f, List.cons(p5_g, List.nil[List[Nat]])))).contains(p5_f)
                                  List.cons(p5_d, List.cons(p5_e, List.cons(p5_f, List.cons(p5_g, List.nil[List[Nat]])))).contains(p5_f)
                                  cons_contains_of_tail_contains(p5_c, List.cons(p5_d, List.cons(p5_e, List.cons(p5_f, List.cons(p5_g, List.nil[List[Nat]])))), p5_f)
                                  List.cons(p5_d, List.cons(p5_e, List.cons(p5_f, List.cons(p5_g, List.nil[List[Nat]])))).contains(p5_f) implies List.cons(p5_c, List.cons(p5_d, List.cons(p5_e, List.cons(p5_f, List.cons(p5_g, List.nil[List[Nat]]))))).contains(p5_f)
                                  List.cons(p5_c, List.cons(p5_d, List.cons(p5_e, List.cons(p5_f, List.cons(p5_g, List.nil[List[Nat]]))))).contains(p5_f)
                                  cons_contains_of_tail_contains(p5_b, List.cons(p5_c, List.cons(p5_d, List.cons(p5_e, List.cons(p5_f, List.cons(p5_g, List.nil[List[Nat]]))))), p5_f)
                                  List.cons(p5_c, List.cons(p5_d, List.cons(p5_e, List.cons(p5_f, List.cons(p5_g, List.nil[List[Nat]]))))).contains(p5_f) implies List.cons(p5_b, List.cons(p5_c, List.cons(p5_d, List.cons(p5_e, List.cons(p5_f, List.cons(p5_g, List.nil[List[Nat]])))))).contains(p5_f)
                                  List.cons(p5_b, List.cons(p5_c, List.cons(p5_d, List.cons(p5_e, List.cons(p5_f, List.cons(p5_g, List.nil[List[Nat]])))))).contains(p5_f)
                                  cons_contains_of_tail_contains(p5_a, List.cons(p5_b, List.cons(p5_c, List.cons(p5_d, List.cons(p5_e, List.cons(p5_f, List.cons(p5_g, List.nil[List[Nat]])))))), p5_f)
                                  List.cons(p5_b, List.cons(p5_c, List.cons(p5_d, List.cons(p5_e, List.cons(p5_f, List.cons(p5_g, List.nil[List[Nat]])))))).contains(p5_f) implies List.cons(p5_a, List.cons(p5_b, List.cons(p5_c, List.cons(p5_d, List.cons(p5_e, List.cons(p5_f, List.cons(p5_g, List.nil[List[Nat]]))))))).contains(p5_f)
                                  List.cons(p5_a, List.cons(p5_b, List.cons(p5_c, List.cons(p5_d, List.cons(p5_e, List.cons(p5_f, List.cons(p5_g, List.nil[List[Nat]]))))))).contains(p5_f)
                                  partitions_five.contains(p5_f)
                                  partitions_five.contains(l)
                                    }
                                    partitions_five.contains(l)
                                }
                                partitions_five.contains(l)
                            }
                            if not a = Nat.2 {
                                if a = Nat.1 {
                                    Nat.1 + sum(t) = Nat.5
                                    add_cancels_left(Nat.1, sum(t), Nat.4)
                                    sum(t) = Nat.4
                                    ni_tail_general(a, t)
                                    non_increasing(List.cons(a, t)) implies non_increasing(t)
                                    non_increasing(List.cons(a, t))
                                    non_increasing(t)
                                    is_partition_intro(t, Nat.4)
                                    sum(t) = Nat.4 and non_increasing(t) and all_positive(t)
                                    is_partition(t, Nat.4)
                                    partition_four_char(t)
                                    is_partition(t, Nat.4) = (t = p4_a or t = p4_b or t = p4_c or t = p4_d or t = p4_e)
                                    t = p4_a or t = p4_b or t = p4_c or t = p4_d or t = p4_e
                                    if t = p4_a {
                                        p4_a = List.cons(Nat.4, List.nil[Nat])
                                        t = List.cons(Nat.4, List.nil[Nat])
                                        cons_contains_head(Nat.4, List.nil[Nat])
                                        List.cons(Nat.4, List.nil[Nat]).contains(Nat.4)
                                        t.contains(Nat.4)
                                        parts_le_head(Nat.1, t, Nat.4)
                                        non_increasing(List.cons(Nat.1, t)) and t.contains(Nat.4) implies Nat.4 <= Nat.1
                                        non_increasing(List.cons(Nat.1, t))
                                        Nat.4 <= Nat.1
                                        lte_imp_not_lt(Nat.4, Nat.1)
                                        Nat.4 <= Nat.1 implies not Nat.1 < Nat.4
                                        not Nat.1 < Nat.4
                                        lt_suc(Nat.1)
                                        Nat.1 < Nat.2
                                        lt_suc(Nat.2)
                                        Nat.2 < Nat.3
                                        lt_suc(Nat.3)
                                        Nat.3 < Nat.4
                                        lt_trans(Nat.2, Nat.3, Nat.4)
                                        Nat.2 < Nat.4
                                        lt_trans(Nat.1, Nat.2, Nat.4)
                                        Nat.1 < Nat.4
                                        false
                                    }
                                    if not t = p4_a {
                                        if t = p4_b {
                                            p4_b = List.cons(Nat.3, List.cons(Nat.1, List.nil[Nat]))
                                            t = List.cons(Nat.3, List.cons(Nat.1, List.nil[Nat]))
                                            cons_contains_head(Nat.3, List.cons(Nat.1, List.nil[Nat]))
                                            List.cons(Nat.3, List.cons(Nat.1, List.nil[Nat])).contains(Nat.3)
                                            t.contains(Nat.3)
                                            parts_le_head(Nat.1, t, Nat.3)
                                            non_increasing(List.cons(Nat.1, t)) and t.contains(Nat.3) implies Nat.3 <= Nat.1
                                            non_increasing(List.cons(Nat.1, t))
                                            Nat.3 <= Nat.1
                                            lte_imp_not_lt(Nat.3, Nat.1)
                                            Nat.3 <= Nat.1 implies not Nat.1 < Nat.3
                                            not Nat.1 < Nat.3
                                            lt_suc(Nat.1)
                                            Nat.1 < Nat.2
                                            lt_suc(Nat.2)
                                            Nat.2 < Nat.3
                                            lt_trans(Nat.1, Nat.2, Nat.3)
                                            Nat.1 < Nat.3
                                            false
                                        }
                                        if not t = p4_b {
                                            if t = p4_c {
                                                p4_c = List.cons(Nat.2, List.cons(Nat.2, List.nil[Nat]))
                                                t = List.cons(Nat.2, List.cons(Nat.2, List.nil[Nat]))
                                                cons_contains_head(Nat.2, List.cons(Nat.2, List.nil[Nat]))
                                                List.cons(Nat.2, List.cons(Nat.2, List.nil[Nat])).contains(Nat.2)
                                                t.contains(Nat.2)
                                                parts_le_head(Nat.1, t, Nat.2)
                                                non_increasing(List.cons(Nat.1, t)) and t.contains(Nat.2) implies Nat.2 <= Nat.1
                                                non_increasing(List.cons(Nat.1, t))
                                                Nat.2 <= Nat.1
                                                lte_imp_not_lt(Nat.2, Nat.1)
                                                Nat.2 <= Nat.1 implies not Nat.1 < Nat.2
                                                not Nat.1 < Nat.2
                                                lt_suc(Nat.1)
                                                Nat.1 < Nat.2
                                                false
                                            }
                                            if not t = p4_c {
                                                if t = p4_d {
                                                    p4_d = List.cons(Nat.2, List.cons(Nat.1, List.cons(Nat.1, List.nil[Nat])))
                                                    t = List.cons(Nat.2, List.cons(Nat.1, List.cons(Nat.1, List.nil[Nat])))
                                                    cons_contains_head(Nat.2, List.cons(Nat.1, List.cons(Nat.1, List.nil[Nat])))
                                                    List.cons(Nat.2, List.cons(Nat.1, List.cons(Nat.1, List.nil[Nat]))).contains(Nat.2)
                                                    t.contains(Nat.2)
                                                    parts_le_head(Nat.1, t, Nat.2)
                                                    non_increasing(List.cons(Nat.1, t)) and t.contains(Nat.2) implies Nat.2 <= Nat.1
                                                    non_increasing(List.cons(Nat.1, t))
                                                    Nat.2 <= Nat.1
                                                    lte_imp_not_lt(Nat.2, Nat.1)
                                                    Nat.2 <= Nat.1 implies not Nat.1 < Nat.2
                                                    not Nat.1 < Nat.2
                                                    lt_suc(Nat.1)
                                                    Nat.1 < Nat.2
                                                    false
                                                }
                                                if not t = p4_d {
                                                    t = p4_e
                                                    p4_e = List.cons(Nat.1, List.cons(Nat.1, List.cons(Nat.1, List.cons(Nat.1, List.nil[Nat]))))
                                                    t = List.cons(Nat.1, List.cons(Nat.1, List.cons(Nat.1, List.cons(Nat.1, List.nil[Nat]))))
                                                    l = List.cons(a, t)
                                                    l = List.cons(Nat.1, List.cons(Nat.1, List.cons(Nat.1, List.cons(Nat.1, List.cons(Nat.1, List.nil[Nat])))))
                                                    p5_g = List.cons(Nat.1, List.cons(Nat.1, List.cons(Nat.1, List.cons(Nat.1, List.cons(Nat.1, List.nil[Nat])))))
                                                    l = p5_g
                                                  partitions_five = List.cons(p5_a, List.cons(p5_b, List.cons(p5_c, List.cons(p5_d, List.cons(p5_e, List.cons(p5_f, List.cons(p5_g, List.nil[List[Nat]])))))))
                                                  cons_contains_head(p5_g, List.nil[List[Nat]])
                                                  List.cons(p5_g, List.nil[List[Nat]]).contains(p5_g)
                                                  cons_contains_of_tail_contains(p5_f, List.cons(p5_g, List.nil[List[Nat]]), p5_g)
                                                  List.cons(p5_g, List.nil[List[Nat]]).contains(p5_g) implies List.cons(p5_f, List.cons(p5_g, List.nil[List[Nat]])).contains(p5_g)
                                                  List.cons(p5_f, List.cons(p5_g, List.nil[List[Nat]])).contains(p5_g)
                                                  cons_contains_of_tail_contains(p5_e, List.cons(p5_f, List.cons(p5_g, List.nil[List[Nat]])), p5_g)
                                                  List.cons(p5_f, List.cons(p5_g, List.nil[List[Nat]])).contains(p5_g) implies List.cons(p5_e, List.cons(p5_f, List.cons(p5_g, List.nil[List[Nat]]))).contains(p5_g)
                                                  List.cons(p5_e, List.cons(p5_f, List.cons(p5_g, List.nil[List[Nat]]))).contains(p5_g)
                                                  cons_contains_of_tail_contains(p5_d, List.cons(p5_e, List.cons(p5_f, List.cons(p5_g, List.nil[List[Nat]]))), p5_g)
                                                  List.cons(p5_e, List.cons(p5_f, List.cons(p5_g, List.nil[List[Nat]]))).contains(p5_g) implies List.cons(p5_d, List.cons(p5_e, List.cons(p5_f, List.cons(p5_g, List.nil[List[Nat]])))).contains(p5_g)
                                                  List.cons(p5_d, List.cons(p5_e, List.cons(p5_f, List.cons(p5_g, List.nil[List[Nat]])))).contains(p5_g)
                                                  cons_contains_of_tail_contains(p5_c, List.cons(p5_d, List.cons(p5_e, List.cons(p5_f, List.cons(p5_g, List.nil[List[Nat]])))), p5_g)
                                                  List.cons(p5_d, List.cons(p5_e, List.cons(p5_f, List.cons(p5_g, List.nil[List[Nat]])))).contains(p5_g) implies List.cons(p5_c, List.cons(p5_d, List.cons(p5_e, List.cons(p5_f, List.cons(p5_g, List.nil[List[Nat]]))))).contains(p5_g)
                                                  List.cons(p5_c, List.cons(p5_d, List.cons(p5_e, List.cons(p5_f, List.cons(p5_g, List.nil[List[Nat]]))))).contains(p5_g)
                                                  cons_contains_of_tail_contains(p5_b, List.cons(p5_c, List.cons(p5_d, List.cons(p5_e, List.cons(p5_f, List.cons(p5_g, List.nil[List[Nat]]))))), p5_g)
                                                  List.cons(p5_c, List.cons(p5_d, List.cons(p5_e, List.cons(p5_f, List.cons(p5_g, List.nil[List[Nat]]))))).contains(p5_g) implies List.cons(p5_b, List.cons(p5_c, List.cons(p5_d, List.cons(p5_e, List.cons(p5_f, List.cons(p5_g, List.nil[List[Nat]])))))).contains(p5_g)
                                                  List.cons(p5_b, List.cons(p5_c, List.cons(p5_d, List.cons(p5_e, List.cons(p5_f, List.cons(p5_g, List.nil[List[Nat]])))))).contains(p5_g)
                                                  cons_contains_of_tail_contains(p5_a, List.cons(p5_b, List.cons(p5_c, List.cons(p5_d, List.cons(p5_e, List.cons(p5_f, List.cons(p5_g, List.nil[List[Nat]])))))), p5_g)
                                                  List.cons(p5_b, List.cons(p5_c, List.cons(p5_d, List.cons(p5_e, List.cons(p5_f, List.cons(p5_g, List.nil[List[Nat]])))))).contains(p5_g) implies List.cons(p5_a, List.cons(p5_b, List.cons(p5_c, List.cons(p5_d, List.cons(p5_e, List.cons(p5_f, List.cons(p5_g, List.nil[List[Nat]]))))))).contains(p5_g)
                                                  List.cons(p5_a, List.cons(p5_b, List.cons(p5_c, List.cons(p5_d, List.cons(p5_e, List.cons(p5_f, List.cons(p5_g, List.nil[List[Nat]]))))))).contains(p5_g)
                                                  partitions_five.contains(p5_g)
                                                  partitions_five.contains(l)
                                                }
                                                partitions_five.contains(l)
                                            }
                                            partitions_five.contains(l)
                                        }
                                        partitions_five.contains(l)
                                    }
                                    partitions_five.contains(l)
                                }
                                if not a = Nat.1 {
                                    a <= Nat.5
                                    lt_imp_lt_suc(a, Nat.5)
                                    a <= Nat.5 implies a < Nat.6
                                    a < Nat.6
                                    lt_six_cases(a)
                                    a = Nat.5 or a = Nat.4 or a = Nat.3 or a = Nat.2 or a = Nat.1 or a = Nat.0
                                    if a = Nat.5 {
                                        not a = Nat.5
                                        false
                                    }
                                    if not a = Nat.5 {
                                        if a = Nat.4 {
                                            not a = Nat.4
                                            false
                                        }
                                        if not a = Nat.4 {
                                            if a = Nat.3 {
                                                not a = Nat.3
                                                false
                                            }
                                            if not a = Nat.3 {
                                                if a = Nat.2 {
                                                    not a = Nat.2
                                                    false
                                                }
                                                if not a = Nat.2 {
                                                    if a = Nat.1 {
                                                        not a = Nat.1
                                                        false
                                                    }
                                                    if not a = Nat.1 {
                                                        a = Nat.0
                                                        Nat.1 <= a
                                                        Nat.1 <= Nat.0
                                                        false
                                                    }
                                                    false
                                                }
                                                false
                                            }
                                            false
                                        }
                                        false
                                    }
                                    partitions_five.contains(l)
                                }
                                partitions_five.contains(l)
                            }
                            partitions_five.contains(l)
                        }
                        partitions_five.contains(l)
                    }
                    partitions_five.contains(l)
                }
                partitions_five.contains(l)
            }
        }
        partitions_five.contains(l)
    }
}
// ---------------------------------------------------------------------------
// The sound direction and the characterization of the partitions of five.
// ---------------------------------------------------------------------------

/// The list 4, 1 is non-increasing.
theorem ni_41 {
    non_increasing(List.cons(Nat.4, List.cons(Nat.1, List.nil[Nat]))) = true
} by {
    non_increasing(List.cons(Nat.4, List.cons(Nat.1, List.nil[Nat]))) =
        non_increasing_from(Nat.4, List.cons(Nat.1, List.nil[Nat]))
    non_increasing_from(Nat.1, List.nil[Nat]) = true
    Nat.1 + Nat.3 = Nat.4
    exists(c: Nat) { Nat.1 + c = Nat.4 }
    Nat.1 <= Nat.4
    non_increasing_from(Nat.4, List.cons(Nat.1, List.nil[Nat])) = true
    non_increasing(List.cons(Nat.4, List.cons(Nat.1, List.nil[Nat]))) = true
}

/// The list 3, 2 is non-increasing.
theorem ni_32 {
    non_increasing(List.cons(Nat.3, List.cons(Nat.2, List.nil[Nat]))) = true
} by {
    non_increasing(List.cons(Nat.3, List.cons(Nat.2, List.nil[Nat]))) =
        non_increasing_from(Nat.3, List.cons(Nat.2, List.nil[Nat]))
    non_increasing_from(Nat.2, List.nil[Nat]) = true
    Nat.2 + Nat.1 = Nat.3
    exists(c: Nat) { Nat.2 + c = Nat.3 }
    Nat.2 <= Nat.3
    non_increasing_from(Nat.3, List.cons(Nat.2, List.nil[Nat])) = true
    non_increasing(List.cons(Nat.3, List.cons(Nat.2, List.nil[Nat]))) = true
}

/// The list 3, 1, 1 is non-increasing.
theorem ni_311 {
    non_increasing(List.cons(Nat.3, List.cons(Nat.1, List.cons(Nat.1, List.nil[Nat])))) = true
} by {
    non_increasing(List.cons(Nat.3, List.cons(Nat.1, List.cons(Nat.1, List.nil[Nat])))) =
        non_increasing_from(Nat.3, List.cons(Nat.1, List.cons(Nat.1, List.nil[Nat])))
    non_increasing_from(Nat.1, List.cons(Nat.1, List.nil[Nat])) =
        (Nat.1 <= Nat.1 and non_increasing_from(Nat.1, List.nil[Nat]))
    non_increasing_from(Nat.1, List.nil[Nat]) = true
    lte_ref(Nat.1)
    Nat.1 <= Nat.1
    non_increasing_from(Nat.1, List.cons(Nat.1, List.nil[Nat])) = true
    lte_ref(Nat.1)
    Nat.1 <= Nat.1
    non_increasing_from(Nat.1, List.cons(Nat.1, List.cons(Nat.1, List.nil[Nat]))) =
        (Nat.1 <= Nat.1 and non_increasing_from(Nat.1, List.cons(Nat.1, List.nil[Nat])))
    non_increasing_from(Nat.1, List.cons(Nat.1, List.cons(Nat.1, List.nil[Nat]))) = true
    lte_ref(Nat.3)
    Nat.3 <= Nat.3
    Nat.1 + Nat.2 = Nat.3
    exists(c: Nat) { Nat.1 + c = Nat.3 }
    Nat.1 <= Nat.3
    non_increasing_from(Nat.3, List.cons(Nat.1, List.cons(Nat.1, List.nil[Nat]))) =
        (Nat.1 <= Nat.3 and non_increasing_from(Nat.1, List.cons(Nat.1, List.nil[Nat])))
    non_increasing_from(Nat.3, List.cons(Nat.1, List.cons(Nat.1, List.nil[Nat]))) = true
    non_increasing(List.cons(Nat.3, List.cons(Nat.1, List.cons(Nat.1, List.nil[Nat])))) = true
}

/// The list 2, 2, 1 is non-increasing.
theorem ni_221 {
    non_increasing(List.cons(Nat.2, List.cons(Nat.2, List.cons(Nat.1, List.nil[Nat])))) = true
} by {
    non_increasing(List.cons(Nat.2, List.cons(Nat.2, List.cons(Nat.1, List.nil[Nat])))) =
        non_increasing_from(Nat.2, List.cons(Nat.2, List.cons(Nat.1, List.nil[Nat])))
    non_increasing_from(Nat.2, List.cons(Nat.1, List.nil[Nat])) =
        (Nat.1 <= Nat.2 and non_increasing_from(Nat.1, List.nil[Nat]))
    non_increasing_from(Nat.1, List.nil[Nat]) = true
    Nat.1 + Nat.1 = Nat.2
    exists(c: Nat) { Nat.1 + c = Nat.2 }
    Nat.1 <= Nat.2
    non_increasing_from(Nat.2, List.cons(Nat.1, List.nil[Nat])) = true
    lte_ref(Nat.2)
    Nat.2 <= Nat.2
    non_increasing_from(Nat.2, List.cons(Nat.2, List.cons(Nat.1, List.nil[Nat]))) =
        (Nat.2 <= Nat.2 and non_increasing_from(Nat.2, List.cons(Nat.1, List.nil[Nat])))
    non_increasing_from(Nat.2, List.cons(Nat.2, List.cons(Nat.1, List.nil[Nat]))) = true
    non_increasing(List.cons(Nat.2, List.cons(Nat.2, List.cons(Nat.1, List.nil[Nat])))) = true
}

/// The list 2, 1, 1, 1 is non-increasing.
theorem ni_2111 {
    non_increasing(List.cons(Nat.2, List.cons(Nat.1, List.cons(Nat.1, List.cons(Nat.1, List.nil[Nat]))))) = true
} by {
    non_increasing(List.cons(Nat.2, List.cons(Nat.1, List.cons(Nat.1, List.cons(Nat.1, List.nil[Nat]))))) =
        non_increasing_from(Nat.2, List.cons(Nat.1, List.cons(Nat.1, List.cons(Nat.1, List.nil[Nat]))))
    non_increasing_from(Nat.1, List.cons(Nat.1, List.nil[Nat])) =
        (Nat.1 <= Nat.1 and non_increasing_from(Nat.1, List.nil[Nat]))
    non_increasing_from(Nat.1, List.nil[Nat]) = true
    lte_ref(Nat.1)
    Nat.1 <= Nat.1
    non_increasing_from(Nat.1, List.cons(Nat.1, List.nil[Nat])) = true
    lte_ref(Nat.1)
    Nat.1 <= Nat.1
    non_increasing_from(Nat.1, List.cons(Nat.1, List.cons(Nat.1, List.nil[Nat]))) =
        (Nat.1 <= Nat.1 and non_increasing_from(Nat.1, List.cons(Nat.1, List.nil[Nat])))
    non_increasing_from(Nat.1, List.cons(Nat.1, List.cons(Nat.1, List.nil[Nat]))) = true
    lte_ref(Nat.1)
    Nat.1 <= Nat.1
    non_increasing_from(Nat.1, List.cons(Nat.1, List.cons(Nat.1, List.cons(Nat.1, List.nil[Nat])))) =
        (Nat.1 <= Nat.1 and non_increasing_from(Nat.1, List.cons(Nat.1, List.cons(Nat.1, List.nil[Nat]))))
    non_increasing_from(Nat.1, List.cons(Nat.1, List.cons(Nat.1, List.cons(Nat.1, List.nil[Nat])))) = true
    lte_ref(Nat.2)
    Nat.2 <= Nat.2
    non_increasing_from(Nat.2, List.cons(Nat.1, List.cons(Nat.1, List.cons(Nat.1, List.nil[Nat])))) =
        (Nat.1 <= Nat.2 and non_increasing_from(Nat.1, List.cons(Nat.1, List.cons(Nat.1, List.nil[Nat]))))
    Nat.1 + Nat.1 = Nat.2
    exists(c: Nat) { Nat.1 + c = Nat.2 }
    Nat.1 <= Nat.2
    non_increasing_from(Nat.2, List.cons(Nat.1, List.cons(Nat.1, List.cons(Nat.1, List.nil[Nat])))) = true
    non_increasing(List.cons(Nat.2, List.cons(Nat.1, List.cons(Nat.1, List.cons(Nat.1, List.nil[Nat]))))) = true
}

/// The list 1, 1, 1, 1, 1 is non-increasing.
theorem ni_11111 {
    non_increasing(List.cons(Nat.1, List.cons(Nat.1, List.cons(Nat.1, List.cons(Nat.1, List.cons(Nat.1, List.nil[Nat])))))) = true
} by {
    non_increasing(List.cons(Nat.1, List.cons(Nat.1, List.cons(Nat.1, List.cons(Nat.1, List.cons(Nat.1, List.nil[Nat])))))) =
        non_increasing_from(Nat.1, List.cons(Nat.1, List.cons(Nat.1, List.cons(Nat.1, List.cons(Nat.1, List.nil[Nat])))))
    non_increasing_from(Nat.1, List.cons(Nat.1, List.nil[Nat])) =
        (Nat.1 <= Nat.1 and non_increasing_from(Nat.1, List.nil[Nat]))
    non_increasing_from(Nat.1, List.nil[Nat]) = true
    lte_ref(Nat.1)
    Nat.1 <= Nat.1
    non_increasing_from(Nat.1, List.cons(Nat.1, List.nil[Nat])) = true
    lte_ref(Nat.1)
    Nat.1 <= Nat.1
    non_increasing_from(Nat.1, List.cons(Nat.1, List.cons(Nat.1, List.nil[Nat]))) =
        (Nat.1 <= Nat.1 and non_increasing_from(Nat.1, List.cons(Nat.1, List.nil[Nat])))
    non_increasing_from(Nat.1, List.cons(Nat.1, List.cons(Nat.1, List.nil[Nat]))) = true
    lte_ref(Nat.1)
    Nat.1 <= Nat.1
    non_increasing_from(Nat.1, List.cons(Nat.1, List.cons(Nat.1, List.cons(Nat.1, List.nil[Nat])))) =
        (Nat.1 <= Nat.1 and non_increasing_from(Nat.1, List.cons(Nat.1, List.cons(Nat.1, List.nil[Nat]))))
    non_increasing_from(Nat.1, List.cons(Nat.1, List.cons(Nat.1, List.cons(Nat.1, List.nil[Nat])))) = true
    lte_ref(Nat.1)
    Nat.1 <= Nat.1
    non_increasing_from(Nat.1, List.cons(Nat.1, List.cons(Nat.1, List.cons(Nat.1, List.cons(Nat.1, List.nil[Nat]))))) =
        (Nat.1 <= Nat.1 and non_increasing_from(Nat.1, List.cons(Nat.1, List.cons(Nat.1, List.cons(Nat.1, List.nil[Nat])))))
    non_increasing_from(Nat.1, List.cons(Nat.1, List.cons(Nat.1, List.cons(Nat.1, List.cons(Nat.1, List.nil[Nat]))))) = true
    non_increasing(List.cons(Nat.1, List.cons(Nat.1, List.cons(Nat.1, List.cons(Nat.1, List.cons(Nat.1, List.nil[Nat])))))) = true
}

/// The list 4, 1 is all-positive.
theorem ap_41 {
    all_positive(List.cons(Nat.4, List.cons(Nat.1, List.nil[Nat]))) = true
} by {
    all_positive(List.cons(Nat.1, List.nil[Nat])) = true
    Nat.0 < Nat.4
    all_positive(List.cons(Nat.4, List.cons(Nat.1, List.nil[Nat]))) = true
}

/// The list 3, 2 is all-positive.
theorem ap_32 {
    all_positive(List.cons(Nat.3, List.cons(Nat.2, List.nil[Nat]))) = true
} by {
    all_positive(List.cons(Nat.2, List.nil[Nat])) = true
    Nat.0 < Nat.3
    all_positive(List.cons(Nat.3, List.cons(Nat.2, List.nil[Nat]))) = true
}

/// The list 3, 1, 1 is all-positive.
theorem ap_311 {
    all_positive(List.cons(Nat.3, List.cons(Nat.1, List.cons(Nat.1, List.nil[Nat])))) = true
} by {
    all_positive(List.cons(Nat.1, List.nil[Nat])) = true
    Nat.0 < Nat.1
    all_positive(List.cons(Nat.1, List.cons(Nat.1, List.nil[Nat]))) = true
    Nat.0 < Nat.3
    all_positive(List.cons(Nat.3, List.cons(Nat.1, List.cons(Nat.1, List.nil[Nat])))) = true
}

/// The list 2, 2, 1 is all-positive.
theorem ap_221 {
    all_positive(List.cons(Nat.2, List.cons(Nat.2, List.cons(Nat.1, List.nil[Nat])))) = true
} by {
    all_positive(List.cons(Nat.1, List.nil[Nat])) = true
    Nat.0 < Nat.1
    all_positive(List.cons(Nat.2, List.cons(Nat.1, List.nil[Nat]))) = true
    Nat.0 < Nat.2
    all_positive(List.cons(Nat.2, List.cons(Nat.2, List.cons(Nat.1, List.nil[Nat])))) = true
}

/// The list 2, 1, 1, 1 is all-positive.
theorem ap_2111 {
    all_positive(List.cons(Nat.2, List.cons(Nat.1, List.cons(Nat.1, List.cons(Nat.1, List.nil[Nat]))))) = true
} by {
    all_positive(List.cons(Nat.1, List.nil[Nat])) = true
    Nat.0 < Nat.1
    all_positive(List.cons(Nat.1, List.cons(Nat.1, List.nil[Nat]))) = true
    Nat.0 < Nat.1
    all_positive(List.cons(Nat.1, List.cons(Nat.1, List.cons(Nat.1, List.nil[Nat])))) = true
    Nat.0 < Nat.2
    all_positive(List.cons(Nat.2, List.cons(Nat.1, List.cons(Nat.1, List.cons(Nat.1, List.nil[Nat]))))) = true
}

/// The list 1, 1, 1, 1, 1 is all-positive.
theorem ap_11111 {
    all_positive(List.cons(Nat.1, List.cons(Nat.1, List.cons(Nat.1, List.cons(Nat.1, List.cons(Nat.1, List.nil[Nat])))))) = true
} by {
    all_positive(List.cons(Nat.1, List.nil[Nat])) = true
    Nat.0 < Nat.1
    all_positive(List.cons(Nat.1, List.cons(Nat.1, List.nil[Nat]))) = true
    Nat.0 < Nat.1
    all_positive(List.cons(Nat.1, List.cons(Nat.1, List.cons(Nat.1, List.nil[Nat])))) = true
    Nat.0 < Nat.1
    all_positive(List.cons(Nat.1, List.cons(Nat.1, List.cons(Nat.1, List.cons(Nat.1, List.nil[Nat]))))) = true
    Nat.0 < Nat.1
    all_positive(List.cons(Nat.1, List.cons(Nat.1, List.cons(Nat.1, List.cons(Nat.1, List.cons(Nat.1, List.nil[Nat])))))) = true
}

/// The or-form of the partitions of five implies being a partition of five.
theorem partition_five_sound(l: List[Nat]) {
    (l = p5_a or l = p5_b or l = p5_c or l = p5_d or l = p5_e or l = p5_f or l = p5_g) implies is_partition(l, Nat.5)
} by {
    if l = p5_a or l = p5_b or l = p5_c or l = p5_d or l = p5_e or l = p5_f or l = p5_g {
        p5_a = List.cons(Nat.5, List.nil[Nat])
        p5_b = List.cons(Nat.4, List.cons(Nat.1, List.nil[Nat]))
        p5_c = List.cons(Nat.3, List.cons(Nat.2, List.nil[Nat]))
        p5_d = List.cons(Nat.3, List.cons(Nat.1, List.cons(Nat.1, List.nil[Nat])))
        p5_e = List.cons(Nat.2, List.cons(Nat.2, List.cons(Nat.1, List.nil[Nat])))
        p5_f = List.cons(Nat.2, List.cons(Nat.1, List.cons(Nat.1, List.cons(Nat.1, List.nil[Nat]))))
        p5_g = List.cons(Nat.1, List.cons(Nat.1, List.cons(Nat.1, List.cons(Nat.1, List.cons(Nat.1, List.nil[Nat])))))
        if l = p5_a {
            l = List.cons(Nat.5, List.nil[Nat])
            sum(List.cons(Nat.5, List.nil[Nat])) = Nat.5 + sum(List.nil[Nat])
            sum(List.nil[Nat]) = Nat.0
            Nat.5 + Nat.0 = Nat.5
            sum(List.cons(Nat.5, List.nil[Nat])) = Nat.5
            ni_single(Nat.5)
            non_increasing(List.cons(Nat.5, List.nil[Nat])) = true
            Nat.0 < Nat.5
            ap_single(Nat.5)
            all_positive(List.cons(Nat.5, List.nil[Nat])) = true
            is_partition_intro(List.cons(Nat.5, List.nil[Nat]), Nat.5)
            sum(List.cons(Nat.5, List.nil[Nat])) = Nat.5 and non_increasing(List.cons(Nat.5, List.nil[Nat])) and all_positive(List.cons(Nat.5, List.nil[Nat]))
            is_partition(List.cons(Nat.5, List.nil[Nat]), Nat.5)
            is_partition(l, Nat.5)
        }
        if not l = p5_a {
            if l = p5_b {
                l = List.cons(Nat.4, List.cons(Nat.1, List.nil[Nat]))
                sum(List.cons(Nat.4, List.cons(Nat.1, List.nil[Nat]))) =
                    Nat.4 + sum(List.cons(Nat.1, List.nil[Nat]))
                sum(List.cons(Nat.1, List.nil[Nat])) = Nat.1 + sum(List.nil[Nat])
                sum(List.nil[Nat]) = Nat.0
                Nat.1 + Nat.0 = Nat.1
                sum(List.cons(Nat.1, List.nil[Nat])) = Nat.1
                Nat.4 + Nat.1 = Nat.5
                sum(List.cons(Nat.4, List.cons(Nat.1, List.nil[Nat]))) = Nat.5
                ni_41
                non_increasing(List.cons(Nat.4, List.cons(Nat.1, List.nil[Nat]))) = true
                ap_41
                all_positive(List.cons(Nat.4, List.cons(Nat.1, List.nil[Nat]))) = true
                is_partition_intro(List.cons(Nat.4, List.cons(Nat.1, List.nil[Nat])), Nat.5)
                sum(List.cons(Nat.4, List.cons(Nat.1, List.nil[Nat]))) = Nat.5 and non_increasing(List.cons(Nat.4, List.cons(Nat.1, List.nil[Nat]))) and all_positive(List.cons(Nat.4, List.cons(Nat.1, List.nil[Nat])))
                is_partition(List.cons(Nat.4, List.cons(Nat.1, List.nil[Nat])), Nat.5)
                is_partition(l, Nat.5)
            }
            if not l = p5_b {
                if l = p5_c {
                    l = List.cons(Nat.3, List.cons(Nat.2, List.nil[Nat]))
                    sum(List.cons(Nat.3, List.cons(Nat.2, List.nil[Nat]))) =
                        Nat.3 + sum(List.cons(Nat.2, List.nil[Nat]))
                    sum(List.cons(Nat.2, List.nil[Nat])) = Nat.2 + sum(List.nil[Nat])
                    sum(List.nil[Nat]) = Nat.0
                    Nat.2 + Nat.0 = Nat.2
                    sum(List.cons(Nat.2, List.nil[Nat])) = Nat.2
                    Nat.3 + Nat.2 = Nat.5
                    sum(List.cons(Nat.3, List.cons(Nat.2, List.nil[Nat]))) = Nat.5
                    ni_32
                    non_increasing(List.cons(Nat.3, List.cons(Nat.2, List.nil[Nat]))) = true
                    ap_32
                    all_positive(List.cons(Nat.3, List.cons(Nat.2, List.nil[Nat]))) = true
                    is_partition_intro(List.cons(Nat.3, List.cons(Nat.2, List.nil[Nat])), Nat.5)
                    sum(List.cons(Nat.3, List.cons(Nat.2, List.nil[Nat]))) = Nat.5 and non_increasing(List.cons(Nat.3, List.cons(Nat.2, List.nil[Nat]))) and all_positive(List.cons(Nat.3, List.cons(Nat.2, List.nil[Nat])))
                    is_partition(List.cons(Nat.3, List.cons(Nat.2, List.nil[Nat])), Nat.5)
                    is_partition(l, Nat.5)
                }
                if not l = p5_c {
                    if l = p5_d {
                        l = List.cons(Nat.3, List.cons(Nat.1, List.cons(Nat.1, List.nil[Nat])))
                        sum(List.cons(Nat.3, List.cons(Nat.1, List.cons(Nat.1, List.nil[Nat])))) =
                            Nat.3 + sum(List.cons(Nat.1, List.cons(Nat.1, List.nil[Nat])))
                        sum(List.cons(Nat.1, List.cons(Nat.1, List.nil[Nat]))) =
                            Nat.1 + sum(List.cons(Nat.1, List.nil[Nat]))
                        sum(List.cons(Nat.1, List.nil[Nat])) = Nat.1 + sum(List.nil[Nat])
                        sum(List.nil[Nat]) = Nat.0
                        Nat.1 + Nat.0 = Nat.1
                        sum(List.cons(Nat.1, List.nil[Nat])) = Nat.1
                        Nat.1 + Nat.1 = Nat.2
                        sum(List.cons(Nat.1, List.cons(Nat.1, List.nil[Nat]))) = Nat.2
                        Nat.3 + Nat.2 = Nat.5
                        sum(List.cons(Nat.3, List.cons(Nat.1, List.cons(Nat.1, List.nil[Nat])))) = Nat.5
                        ni_311
                        non_increasing(List.cons(Nat.3, List.cons(Nat.1, List.cons(Nat.1, List.nil[Nat])))) = true
                        ap_311
                        all_positive(List.cons(Nat.3, List.cons(Nat.1, List.cons(Nat.1, List.nil[Nat])))) = true
                        is_partition_intro(List.cons(Nat.3, List.cons(Nat.1, List.cons(Nat.1, List.nil[Nat]))), Nat.5)
                        sum(List.cons(Nat.3, List.cons(Nat.1, List.cons(Nat.1, List.nil[Nat])))) = Nat.5 and non_increasing(List.cons(Nat.3, List.cons(Nat.1, List.cons(Nat.1, List.nil[Nat])))) and all_positive(List.cons(Nat.3, List.cons(Nat.1, List.cons(Nat.1, List.nil[Nat]))))
                        is_partition(List.cons(Nat.3, List.cons(Nat.1, List.cons(Nat.1, List.nil[Nat]))), Nat.5)
                        is_partition(l, Nat.5)
                    }
                    if not l = p5_d {
                        if l = p5_e {
                            l = List.cons(Nat.2, List.cons(Nat.2, List.cons(Nat.1, List.nil[Nat])))
                            sum(List.cons(Nat.2, List.cons(Nat.2, List.cons(Nat.1, List.nil[Nat])))) =
                                Nat.2 + sum(List.cons(Nat.2, List.cons(Nat.1, List.nil[Nat])))
                            sum(List.cons(Nat.2, List.cons(Nat.1, List.nil[Nat]))) =
                                Nat.2 + sum(List.cons(Nat.1, List.nil[Nat]))
                            sum(List.cons(Nat.1, List.nil[Nat])) = Nat.1 + sum(List.nil[Nat])
                            sum(List.nil[Nat]) = Nat.0
                            Nat.1 + Nat.0 = Nat.1
                            sum(List.cons(Nat.1, List.nil[Nat])) = Nat.1
                            Nat.2 + Nat.1 = Nat.3
                            sum(List.cons(Nat.2, List.cons(Nat.1, List.nil[Nat]))) = Nat.3
                            Nat.2 + Nat.3 = Nat.5
                            sum(List.cons(Nat.2, List.cons(Nat.2, List.cons(Nat.1, List.nil[Nat])))) = Nat.5
                            ni_221
                            non_increasing(List.cons(Nat.2, List.cons(Nat.2, List.cons(Nat.1, List.nil[Nat])))) = true
                            ap_221
                            all_positive(List.cons(Nat.2, List.cons(Nat.2, List.cons(Nat.1, List.nil[Nat])))) = true
                            is_partition_intro(List.cons(Nat.2, List.cons(Nat.2, List.cons(Nat.1, List.nil[Nat]))), Nat.5)
                            sum(List.cons(Nat.2, List.cons(Nat.2, List.cons(Nat.1, List.nil[Nat])))) = Nat.5 and non_increasing(List.cons(Nat.2, List.cons(Nat.2, List.cons(Nat.1, List.nil[Nat])))) and all_positive(List.cons(Nat.2, List.cons(Nat.2, List.cons(Nat.1, List.nil[Nat]))))
                            is_partition(List.cons(Nat.2, List.cons(Nat.2, List.cons(Nat.1, List.nil[Nat]))), Nat.5)
                            is_partition(l, Nat.5)
                        }
                        if not l = p5_e {
                            if l = p5_f {
                                l = List.cons(Nat.2, List.cons(Nat.1, List.cons(Nat.1, List.cons(Nat.1, List.nil[Nat]))))
                                sum(List.cons(Nat.2, List.cons(Nat.1, List.cons(Nat.1, List.cons(Nat.1, List.nil[Nat]))))) =
                                    Nat.2 + sum(List.cons(Nat.1, List.cons(Nat.1, List.cons(Nat.1, List.nil[Nat]))))
                                sum(List.cons(Nat.1, List.cons(Nat.1, List.cons(Nat.1, List.nil[Nat])))) =
                                    Nat.1 + sum(List.cons(Nat.1, List.cons(Nat.1, List.nil[Nat])))
                                sum(List.cons(Nat.1, List.cons(Nat.1, List.nil[Nat]))) =
                                    Nat.1 + sum(List.cons(Nat.1, List.nil[Nat]))
                                sum(List.cons(Nat.1, List.nil[Nat])) = Nat.1 + sum(List.nil[Nat])
                                sum(List.nil[Nat]) = Nat.0
                                Nat.1 + Nat.0 = Nat.1
                                sum(List.cons(Nat.1, List.nil[Nat])) = Nat.1
                                Nat.1 + Nat.1 = Nat.2
                                sum(List.cons(Nat.1, List.cons(Nat.1, List.nil[Nat]))) = Nat.2
                                Nat.1 + Nat.2 = Nat.3
                                sum(List.cons(Nat.1, List.cons(Nat.1, List.cons(Nat.1, List.nil[Nat])))) = Nat.3
                                Nat.2 + Nat.3 = Nat.5
                                sum(List.cons(Nat.2, List.cons(Nat.1, List.cons(Nat.1, List.cons(Nat.1, List.nil[Nat]))))) = Nat.5
                                ni_2111
                                non_increasing(List.cons(Nat.2, List.cons(Nat.1, List.cons(Nat.1, List.cons(Nat.1, List.nil[Nat]))))) = true
                                ap_2111
                                all_positive(List.cons(Nat.2, List.cons(Nat.1, List.cons(Nat.1, List.cons(Nat.1, List.nil[Nat]))))) = true
                                is_partition_intro(List.cons(Nat.2, List.cons(Nat.1, List.cons(Nat.1, List.cons(Nat.1, List.nil[Nat])))), Nat.5)
                                sum(List.cons(Nat.2, List.cons(Nat.1, List.cons(Nat.1, List.cons(Nat.1, List.nil[Nat]))))) = Nat.5 and non_increasing(List.cons(Nat.2, List.cons(Nat.1, List.cons(Nat.1, List.cons(Nat.1, List.nil[Nat]))))) and all_positive(List.cons(Nat.2, List.cons(Nat.1, List.cons(Nat.1, List.cons(Nat.1, List.nil[Nat])))))
                                is_partition(List.cons(Nat.2, List.cons(Nat.1, List.cons(Nat.1, List.cons(Nat.1, List.nil[Nat])))), Nat.5)
                                is_partition(l, Nat.5)
                            }
                            if not l = p5_f {
                                l = p5_g
                                l = List.cons(Nat.1, List.cons(Nat.1, List.cons(Nat.1, List.cons(Nat.1, List.cons(Nat.1, List.nil[Nat])))))
                                sum(List.cons(Nat.1, List.cons(Nat.1, List.cons(Nat.1, List.cons(Nat.1, List.cons(Nat.1, List.nil[Nat])))))) =
                                    Nat.1 + sum(List.cons(Nat.1, List.cons(Nat.1, List.cons(Nat.1, List.cons(Nat.1, List.nil[Nat])))))
                                sum(List.cons(Nat.1, List.cons(Nat.1, List.cons(Nat.1, List.cons(Nat.1, List.nil[Nat]))))) =
                                    Nat.1 + sum(List.cons(Nat.1, List.cons(Nat.1, List.cons(Nat.1, List.nil[Nat]))))
                                sum(List.cons(Nat.1, List.cons(Nat.1, List.cons(Nat.1, List.nil[Nat])))) =
                                    Nat.1 + sum(List.cons(Nat.1, List.cons(Nat.1, List.nil[Nat])))
                                sum(List.cons(Nat.1, List.cons(Nat.1, List.nil[Nat]))) =
                                    Nat.1 + sum(List.cons(Nat.1, List.nil[Nat]))
                                sum(List.cons(Nat.1, List.nil[Nat])) = Nat.1 + sum(List.nil[Nat])
                                sum(List.nil[Nat]) = Nat.0
                                Nat.1 + Nat.0 = Nat.1
                                sum(List.cons(Nat.1, List.nil[Nat])) = Nat.1
                                Nat.1 + Nat.1 = Nat.2
                                sum(List.cons(Nat.1, List.cons(Nat.1, List.nil[Nat]))) = Nat.2
                                Nat.1 + Nat.2 = Nat.3
                                sum(List.cons(Nat.1, List.cons(Nat.1, List.cons(Nat.1, List.nil[Nat])))) = Nat.3
                                Nat.1 + Nat.3 = Nat.4
                                sum(List.cons(Nat.1, List.cons(Nat.1, List.cons(Nat.1, List.cons(Nat.1, List.nil[Nat]))))) = Nat.4
                                Nat.1 + Nat.4 = Nat.5
                                sum(List.cons(Nat.1, List.cons(Nat.1, List.cons(Nat.1, List.cons(Nat.1, List.cons(Nat.1, List.nil[Nat])))))) = Nat.5
                                ni_11111
                                non_increasing(List.cons(Nat.1, List.cons(Nat.1, List.cons(Nat.1, List.cons(Nat.1, List.cons(Nat.1, List.nil[Nat])))))) = true
                                ap_11111
                                all_positive(List.cons(Nat.1, List.cons(Nat.1, List.cons(Nat.1, List.cons(Nat.1, List.cons(Nat.1, List.nil[Nat])))))) = true
                                is_partition_intro(List.cons(Nat.1, List.cons(Nat.1, List.cons(Nat.1, List.cons(Nat.1, List.cons(Nat.1, List.nil[Nat]))))), Nat.5)
                                sum(List.cons(Nat.1, List.cons(Nat.1, List.cons(Nat.1, List.cons(Nat.1, List.cons(Nat.1, List.nil[Nat])))))) = Nat.5 and non_increasing(List.cons(Nat.1, List.cons(Nat.1, List.cons(Nat.1, List.cons(Nat.1, List.cons(Nat.1, List.nil[Nat])))))) and all_positive(List.cons(Nat.1, List.cons(Nat.1, List.cons(Nat.1, List.cons(Nat.1, List.cons(Nat.1, List.nil[Nat]))))))
                                is_partition(List.cons(Nat.1, List.cons(Nat.1, List.cons(Nat.1, List.cons(Nat.1, List.cons(Nat.1, List.nil[Nat]))))), Nat.5)
                                is_partition(l, Nat.5)
                            }
                            is_partition(l, Nat.5)
                        }
                        is_partition(l, Nat.5)
                    }
                    is_partition(l, Nat.5)
                }
                is_partition(l, Nat.5)
            }
            is_partition(l, Nat.5)
        }
        is_partition(l, Nat.5)
    }
}

/// Membership in the explicit list of partitions of five means being a
/// partition of five.
theorem partitions_five_membership(l: List[Nat]) {
    partitions_five.contains(l) implies is_partition(l, Nat.5)
} by {
    if partitions_five.contains(l) {
        partitions_five = List.cons(p5_a, List.cons(p5_b, List.cons(p5_c, List.cons(p5_d, List.cons(p5_e, List.cons(p5_f, List.cons(p5_g, List.nil[List[Nat]])))))))
        cons_contains_eq(p5_a, List.cons(p5_b, List.cons(p5_c, List.cons(p5_d, List.cons(p5_e, List.cons(p5_f, List.cons(p5_g, List.nil[List[Nat]])))))), l)
        partitions_five.contains(l) =
            (l = p5_a or List.cons(p5_b, List.cons(p5_c, List.cons(p5_d, List.cons(p5_e, List.cons(p5_f, List.cons(p5_g, List.nil[List[Nat]])))))).contains(l))
        cons_contains_eq(p5_b, List.cons(p5_c, List.cons(p5_d, List.cons(p5_e, List.cons(p5_f, List.cons(p5_g, List.nil[List[Nat]]))))), l)
        List.cons(p5_b, List.cons(p5_c, List.cons(p5_d, List.cons(p5_e, List.cons(p5_f, List.cons(p5_g, List.nil[List[Nat]])))))).contains(l) =
            (l = p5_b or List.cons(p5_c, List.cons(p5_d, List.cons(p5_e, List.cons(p5_f, List.cons(p5_g, List.nil[List[Nat]]))))).contains(l))
        cons_contains_eq(p5_c, List.cons(p5_d, List.cons(p5_e, List.cons(p5_f, List.cons(p5_g, List.nil[List[Nat]])))), l)
        List.cons(p5_c, List.cons(p5_d, List.cons(p5_e, List.cons(p5_f, List.cons(p5_g, List.nil[List[Nat]]))))).contains(l) =
            (l = p5_c or List.cons(p5_d, List.cons(p5_e, List.cons(p5_f, List.cons(p5_g, List.nil[List[Nat]])))).contains(l))
        cons_contains_eq(p5_d, List.cons(p5_e, List.cons(p5_f, List.cons(p5_g, List.nil[List[Nat]]))), l)
        List.cons(p5_d, List.cons(p5_e, List.cons(p5_f, List.cons(p5_g, List.nil[List[Nat]])))).contains(l) =
            (l = p5_d or List.cons(p5_e, List.cons(p5_f, List.cons(p5_g, List.nil[List[Nat]]))).contains(l))
        cons_contains_eq(p5_e, List.cons(p5_f, List.cons(p5_g, List.nil[List[Nat]])), l)
        List.cons(p5_e, List.cons(p5_f, List.cons(p5_g, List.nil[List[Nat]]))).contains(l) =
            (l = p5_e or List.cons(p5_f, List.cons(p5_g, List.nil[List[Nat]])).contains(l))
        cons_contains_eq(p5_f, List.cons(p5_g, List.nil[List[Nat]]), l)
        List.cons(p5_f, List.cons(p5_g, List.nil[List[Nat]])).contains(l) =
            (l = p5_f or List.cons(p5_g, List.nil[List[Nat]]).contains(l))
        cons_contains_eq(p5_g, List.nil[List[Nat]], l)
        List.cons(p5_g, List.nil[List[Nat]]).contains(l) =
            (l = p5_g or List.nil[List[Nat]].contains(l))
        nil_not_contains(l)
        l = p5_a or l = p5_b or l = p5_c or l = p5_d or l = p5_e or l = p5_f or l = p5_g
        partition_five_sound(l)
        (l = p5_a or l = p5_b or l = p5_c or l = p5_d or l = p5_e or l = p5_f or l = p5_g) implies is_partition(l, Nat.5)
        is_partition(l, Nat.5)
    }
}

/// The partitions of five are exactly 5, 4 + 1, 3 + 2, 3 + 1 + 1, 2 + 2 + 1,
/// 2 + 1 + 1 + 1 and 1 + 1 + 1 + 1 + 1.
theorem partition_five_char(l: List[Nat]) {
    is_partition(l, Nat.5) = (l = p5_a or l = p5_b or l = p5_c or l = p5_d or l = p5_e or l = p5_f or l = p5_g)
} by {
    partitions_five_contains(l)
    is_partition(l, Nat.5) implies partitions_five.contains(l)
    partitions_five_membership(l)
    partitions_five.contains(l) implies is_partition(l, Nat.5)
    if is_partition(l, Nat.5) {
        partitions_five_contains(l)
        is_partition(l, Nat.5) implies partitions_five.contains(l)
        partitions_five.contains(l)
        partitions_five = List.cons(p5_a, List.cons(p5_b, List.cons(p5_c, List.cons(p5_d, List.cons(p5_e, List.cons(p5_f, List.cons(p5_g, List.nil[List[Nat]])))))))
        cons_contains_eq(p5_a, List.cons(p5_b, List.cons(p5_c, List.cons(p5_d, List.cons(p5_e, List.cons(p5_f, List.cons(p5_g, List.nil[List[Nat]])))))), l)
        partitions_five.contains(l) =
            (l = p5_a or List.cons(p5_b, List.cons(p5_c, List.cons(p5_d, List.cons(p5_e, List.cons(p5_f, List.cons(p5_g, List.nil[List[Nat]])))))).contains(l))
        cons_contains_eq(p5_b, List.cons(p5_c, List.cons(p5_d, List.cons(p5_e, List.cons(p5_f, List.cons(p5_g, List.nil[List[Nat]]))))), l)
        List.cons(p5_b, List.cons(p5_c, List.cons(p5_d, List.cons(p5_e, List.cons(p5_f, List.cons(p5_g, List.nil[List[Nat]])))))).contains(l) =
            (l = p5_b or List.cons(p5_c, List.cons(p5_d, List.cons(p5_e, List.cons(p5_f, List.cons(p5_g, List.nil[List[Nat]]))))).contains(l))
        cons_contains_eq(p5_c, List.cons(p5_d, List.cons(p5_e, List.cons(p5_f, List.cons(p5_g, List.nil[List[Nat]])))), l)
        List.cons(p5_c, List.cons(p5_d, List.cons(p5_e, List.cons(p5_f, List.cons(p5_g, List.nil[List[Nat]]))))).contains(l) =
            (l = p5_c or List.cons(p5_d, List.cons(p5_e, List.cons(p5_f, List.cons(p5_g, List.nil[List[Nat]])))).contains(l))
        cons_contains_eq(p5_d, List.cons(p5_e, List.cons(p5_f, List.cons(p5_g, List.nil[List[Nat]]))), l)
        List.cons(p5_d, List.cons(p5_e, List.cons(p5_f, List.cons(p5_g, List.nil[List[Nat]])))).contains(l) =
            (l = p5_d or List.cons(p5_e, List.cons(p5_f, List.cons(p5_g, List.nil[List[Nat]]))).contains(l))
        cons_contains_eq(p5_e, List.cons(p5_f, List.cons(p5_g, List.nil[List[Nat]])), l)
        List.cons(p5_e, List.cons(p5_f, List.cons(p5_g, List.nil[List[Nat]]))).contains(l) =
            (l = p5_e or List.cons(p5_f, List.cons(p5_g, List.nil[List[Nat]])).contains(l))
        cons_contains_eq(p5_f, List.cons(p5_g, List.nil[List[Nat]]), l)
        List.cons(p5_f, List.cons(p5_g, List.nil[List[Nat]])).contains(l) =
            (l = p5_f or List.cons(p5_g, List.nil[List[Nat]]).contains(l))
        cons_contains_eq(p5_g, List.nil[List[Nat]], l)
        List.cons(p5_g, List.nil[List[Nat]]).contains(l) =
            (l = p5_g or List.nil[List[Nat]].contains(l))
        nil_not_contains(l)
        l = p5_a or l = p5_b or l = p5_c or l = p5_d or l = p5_e or l = p5_f or l = p5_g
        is_partition(l, Nat.5) = (l = p5_a or l = p5_b or l = p5_c or l = p5_d or l = p5_e or l = p5_f or l = p5_g)
    }
    if not is_partition(l, Nat.5) {
        if l = p5_a or l = p5_b or l = p5_c or l = p5_d or l = p5_e or l = p5_f or l = p5_g {
            partition_five_sound(l)
            (l = p5_a or l = p5_b or l = p5_c or l = p5_d or l = p5_e or l = p5_f or l = p5_g) implies is_partition(l, Nat.5)
            is_partition(l, Nat.5)
            false
        }
        not (l = p5_a or l = p5_b or l = p5_c or l = p5_d or l = p5_e or l = p5_f or l = p5_g)
        is_partition(l, Nat.5) = (l = p5_a or l = p5_b or l = p5_c or l = p5_d or l = p5_e or l = p5_f or l = p5_g)
    }
    is_partition(l, Nat.5) = (l = p5_a or l = p5_b or l = p5_c or l = p5_d or l = p5_e or l = p5_f or l = p5_g)
}

// ---------------------------------------------------------------------------

// ---------------------------------------------------------------------------
// The partitions of five are seven in number: p(5) = 7.
// ---------------------------------------------------------------------------

/// The partition p5_a does not occur among the later partitions.
theorem not_contains_p5_a {
    not List.cons(p5_b, List.cons(p5_c, List.cons(p5_d, List.cons(p5_e, List.cons(p5_f, List.cons(p5_g, List.nil[List[Nat]])))))).contains(p5_a)
} by {
    if List.cons(p5_b, List.cons(p5_c, List.cons(p5_d, List.cons(p5_e, List.cons(p5_f, List.cons(p5_g, List.nil[List[Nat]])))))).contains(p5_a) {
        cons_contains_eq(p5_b, List.cons(p5_c, List.cons(p5_d, List.cons(p5_e, List.cons(p5_f, List.cons(p5_g, List.nil[List[Nat]]))))), p5_a)
        List.cons(p5_b, List.cons(p5_c, List.cons(p5_d, List.cons(p5_e, List.cons(p5_f, List.cons(p5_g, List.nil[List[Nat]])))))).contains(p5_a) =
            (p5_a = p5_b or List.cons(p5_c, List.cons(p5_d, List.cons(p5_e, List.cons(p5_f, List.cons(p5_g, List.nil[List[Nat]]))))).contains(p5_a))
        cons_contains_eq(p5_c, List.cons(p5_d, List.cons(p5_e, List.cons(p5_f, List.cons(p5_g, List.nil[List[Nat]])))), p5_a)
        List.cons(p5_c, List.cons(p5_d, List.cons(p5_e, List.cons(p5_f, List.cons(p5_g, List.nil[List[Nat]]))))).contains(p5_a) =
            (p5_a = p5_c or List.cons(p5_d, List.cons(p5_e, List.cons(p5_f, List.cons(p5_g, List.nil[List[Nat]])))).contains(p5_a))
        cons_contains_eq(p5_d, List.cons(p5_e, List.cons(p5_f, List.cons(p5_g, List.nil[List[Nat]]))), p5_a)
        List.cons(p5_d, List.cons(p5_e, List.cons(p5_f, List.cons(p5_g, List.nil[List[Nat]])))).contains(p5_a) =
            (p5_a = p5_d or List.cons(p5_e, List.cons(p5_f, List.cons(p5_g, List.nil[List[Nat]]))).contains(p5_a))
        cons_contains_eq(p5_e, List.cons(p5_f, List.cons(p5_g, List.nil[List[Nat]])), p5_a)
        List.cons(p5_e, List.cons(p5_f, List.cons(p5_g, List.nil[List[Nat]]))).contains(p5_a) =
            (p5_a = p5_e or List.cons(p5_f, List.cons(p5_g, List.nil[List[Nat]])).contains(p5_a))
        cons_contains_eq(p5_f, List.cons(p5_g, List.nil[List[Nat]]), p5_a)
        List.cons(p5_f, List.cons(p5_g, List.nil[List[Nat]])).contains(p5_a) =
            (p5_a = p5_f or List.cons(p5_g, List.nil[List[Nat]]).contains(p5_a))
        cons_contains_eq(p5_g, List.nil[List[Nat]], p5_a)
        List.cons(p5_g, List.nil[List[Nat]]).contains(p5_a) =
            (p5_a = p5_g or List.nil[List[Nat]].contains(p5_a))
        nil_not_contains(p5_a)
        p5_a = p5_b or p5_a = p5_c or p5_a = p5_d or p5_a = p5_e or p5_a = p5_f or p5_a = p5_g
            if p5_a = p5_b {
                p5_a = List.cons(Nat.5, List.nil[Nat])
                p5_b = List.cons(Nat.4, List.cons(Nat.1, List.nil[Nat]))
                List.cons(Nat.5, List.nil[Nat]) = List.cons(Nat.4, List.cons(Nat.1, List.nil[Nat]))
                false
            }
            if not p5_a = p5_b {
                if p5_a = p5_c {
                    p5_a = List.cons(Nat.5, List.nil[Nat])
                    p5_c = List.cons(Nat.3, List.cons(Nat.2, List.nil[Nat]))
                    List.cons(Nat.5, List.nil[Nat]) = List.cons(Nat.3, List.cons(Nat.2, List.nil[Nat]))
                    false
                }
                if not p5_a = p5_c {
                    if p5_a = p5_d {
                        p5_a = List.cons(Nat.5, List.nil[Nat])
                        p5_d = List.cons(Nat.3, List.cons(Nat.1, List.cons(Nat.1, List.nil[Nat])))
                        List.cons(Nat.5, List.nil[Nat]) = List.cons(Nat.3, List.cons(Nat.1, List.cons(Nat.1, List.nil[Nat])))
                        false
                    }
                    if not p5_a = p5_d {
                        if p5_a = p5_e {
                            p5_a = List.cons(Nat.5, List.nil[Nat])
                            p5_e = List.cons(Nat.2, List.cons(Nat.2, List.cons(Nat.1, List.nil[Nat])))
                            List.cons(Nat.5, List.nil[Nat]) = List.cons(Nat.2, List.cons(Nat.2, List.cons(Nat.1, List.nil[Nat])))
                            false
                        }
                        if not p5_a = p5_e {
                            if p5_a = p5_f {
                                p5_a = List.cons(Nat.5, List.nil[Nat])
                                p5_f = List.cons(Nat.2, List.cons(Nat.1, List.cons(Nat.1, List.cons(Nat.1, List.nil[Nat]))))
                                List.cons(Nat.5, List.nil[Nat]) = List.cons(Nat.2, List.cons(Nat.1, List.cons(Nat.1, List.cons(Nat.1, List.nil[Nat]))))
                                false
                            }
                            if not p5_a = p5_f {
                                if p5_a = p5_g {
                                    p5_a = List.cons(Nat.5, List.nil[Nat])
                                    p5_g = List.cons(Nat.1, List.cons(Nat.1, List.cons(Nat.1, List.cons(Nat.1, List.cons(Nat.1, List.nil[Nat])))))
                                    List.cons(Nat.5, List.nil[Nat]) = List.cons(Nat.1, List.cons(Nat.1, List.cons(Nat.1, List.cons(Nat.1, List.cons(Nat.1, List.nil[Nat])))))
                                    false
                                }
                            }
                            false
                        }
                        false
                    }
                    false
                }
                false
            }
            false
    }
    not List.cons(p5_b, List.cons(p5_c, List.cons(p5_d, List.cons(p5_e, List.cons(p5_f, List.cons(p5_g, List.nil[List[Nat]])))))).contains(p5_a)
}

/// The partition p5_b does not occur among the later partitions.
theorem not_contains_p5_b {
    not List.cons(p5_c, List.cons(p5_d, List.cons(p5_e, List.cons(p5_f, List.cons(p5_g, List.nil[List[Nat]]))))).contains(p5_b)
} by {
    if List.cons(p5_c, List.cons(p5_d, List.cons(p5_e, List.cons(p5_f, List.cons(p5_g, List.nil[List[Nat]]))))).contains(p5_b) {
        cons_contains_eq(p5_c, List.cons(p5_d, List.cons(p5_e, List.cons(p5_f, List.cons(p5_g, List.nil[List[Nat]])))), p5_b)
        List.cons(p5_c, List.cons(p5_d, List.cons(p5_e, List.cons(p5_f, List.cons(p5_g, List.nil[List[Nat]]))))).contains(p5_b) =
            (p5_b = p5_c or List.cons(p5_d, List.cons(p5_e, List.cons(p5_f, List.cons(p5_g, List.nil[List[Nat]])))).contains(p5_b))
        cons_contains_eq(p5_d, List.cons(p5_e, List.cons(p5_f, List.cons(p5_g, List.nil[List[Nat]]))), p5_b)
        List.cons(p5_d, List.cons(p5_e, List.cons(p5_f, List.cons(p5_g, List.nil[List[Nat]])))).contains(p5_b) =
            (p5_b = p5_d or List.cons(p5_e, List.cons(p5_f, List.cons(p5_g, List.nil[List[Nat]]))).contains(p5_b))
        cons_contains_eq(p5_e, List.cons(p5_f, List.cons(p5_g, List.nil[List[Nat]])), p5_b)
        List.cons(p5_e, List.cons(p5_f, List.cons(p5_g, List.nil[List[Nat]]))).contains(p5_b) =
            (p5_b = p5_e or List.cons(p5_f, List.cons(p5_g, List.nil[List[Nat]])).contains(p5_b))
        cons_contains_eq(p5_f, List.cons(p5_g, List.nil[List[Nat]]), p5_b)
        List.cons(p5_f, List.cons(p5_g, List.nil[List[Nat]])).contains(p5_b) =
            (p5_b = p5_f or List.cons(p5_g, List.nil[List[Nat]]).contains(p5_b))
        cons_contains_eq(p5_g, List.nil[List[Nat]], p5_b)
        List.cons(p5_g, List.nil[List[Nat]]).contains(p5_b) =
            (p5_b = p5_g or List.nil[List[Nat]].contains(p5_b))
        nil_not_contains(p5_b)
        p5_b = p5_c or p5_b = p5_d or p5_b = p5_e or p5_b = p5_f or p5_b = p5_g
            if p5_b = p5_c {
                p5_b = List.cons(Nat.4, List.cons(Nat.1, List.nil[Nat]))
                p5_c = List.cons(Nat.3, List.cons(Nat.2, List.nil[Nat]))
                List.cons(Nat.4, List.cons(Nat.1, List.nil[Nat])) = List.cons(Nat.3, List.cons(Nat.2, List.nil[Nat]))
                false
            }
            if not p5_b = p5_c {
                if p5_b = p5_d {
                    p5_b = List.cons(Nat.4, List.cons(Nat.1, List.nil[Nat]))
                    p5_d = List.cons(Nat.3, List.cons(Nat.1, List.cons(Nat.1, List.nil[Nat])))
                    List.cons(Nat.4, List.cons(Nat.1, List.nil[Nat])) = List.cons(Nat.3, List.cons(Nat.1, List.cons(Nat.1, List.nil[Nat])))
                    false
                }
                if not p5_b = p5_d {
                    if p5_b = p5_e {
                        p5_b = List.cons(Nat.4, List.cons(Nat.1, List.nil[Nat]))
                        p5_e = List.cons(Nat.2, List.cons(Nat.2, List.cons(Nat.1, List.nil[Nat])))
                        List.cons(Nat.4, List.cons(Nat.1, List.nil[Nat])) = List.cons(Nat.2, List.cons(Nat.2, List.cons(Nat.1, List.nil[Nat])))
                        false
                    }
                    if not p5_b = p5_e {
                        if p5_b = p5_f {
                            p5_b = List.cons(Nat.4, List.cons(Nat.1, List.nil[Nat]))
                            p5_f = List.cons(Nat.2, List.cons(Nat.1, List.cons(Nat.1, List.cons(Nat.1, List.nil[Nat]))))
                            List.cons(Nat.4, List.cons(Nat.1, List.nil[Nat])) = List.cons(Nat.2, List.cons(Nat.1, List.cons(Nat.1, List.cons(Nat.1, List.nil[Nat]))))
                            false
                        }
                        if not p5_b = p5_f {
                            if p5_b = p5_g {
                                p5_b = List.cons(Nat.4, List.cons(Nat.1, List.nil[Nat]))
                                p5_g = List.cons(Nat.1, List.cons(Nat.1, List.cons(Nat.1, List.cons(Nat.1, List.cons(Nat.1, List.nil[Nat])))))
                                List.cons(Nat.4, List.cons(Nat.1, List.nil[Nat])) = List.cons(Nat.1, List.cons(Nat.1, List.cons(Nat.1, List.cons(Nat.1, List.cons(Nat.1, List.nil[Nat])))))
                                false
                            }
                        }
                        false
                    }
                    false
                }
                false
            }
            false
    }
    not List.cons(p5_c, List.cons(p5_d, List.cons(p5_e, List.cons(p5_f, List.cons(p5_g, List.nil[List[Nat]]))))).contains(p5_b)
}

/// The partition p5_c does not occur among the later partitions.
theorem not_contains_p5_c {
    not List.cons(p5_d, List.cons(p5_e, List.cons(p5_f, List.cons(p5_g, List.nil[List[Nat]])))).contains(p5_c)
} by {
    if List.cons(p5_d, List.cons(p5_e, List.cons(p5_f, List.cons(p5_g, List.nil[List[Nat]])))).contains(p5_c) {
        cons_contains_eq(p5_d, List.cons(p5_e, List.cons(p5_f, List.cons(p5_g, List.nil[List[Nat]]))), p5_c)
        List.cons(p5_d, List.cons(p5_e, List.cons(p5_f, List.cons(p5_g, List.nil[List[Nat]])))).contains(p5_c) =
            (p5_c = p5_d or List.cons(p5_e, List.cons(p5_f, List.cons(p5_g, List.nil[List[Nat]]))).contains(p5_c))
        cons_contains_eq(p5_e, List.cons(p5_f, List.cons(p5_g, List.nil[List[Nat]])), p5_c)
        List.cons(p5_e, List.cons(p5_f, List.cons(p5_g, List.nil[List[Nat]]))).contains(p5_c) =
            (p5_c = p5_e or List.cons(p5_f, List.cons(p5_g, List.nil[List[Nat]])).contains(p5_c))
        cons_contains_eq(p5_f, List.cons(p5_g, List.nil[List[Nat]]), p5_c)
        List.cons(p5_f, List.cons(p5_g, List.nil[List[Nat]])).contains(p5_c) =
            (p5_c = p5_f or List.cons(p5_g, List.nil[List[Nat]]).contains(p5_c))
        cons_contains_eq(p5_g, List.nil[List[Nat]], p5_c)
        List.cons(p5_g, List.nil[List[Nat]]).contains(p5_c) =
            (p5_c = p5_g or List.nil[List[Nat]].contains(p5_c))
        nil_not_contains(p5_c)
        p5_c = p5_d or p5_c = p5_e or p5_c = p5_f or p5_c = p5_g
            if p5_c = p5_d {
                p5_c = List.cons(Nat.3, List.cons(Nat.2, List.nil[Nat]))
                p5_d = List.cons(Nat.3, List.cons(Nat.1, List.cons(Nat.1, List.nil[Nat])))
                List.cons(Nat.3, List.cons(Nat.2, List.nil[Nat])) = List.cons(Nat.3, List.cons(Nat.1, List.cons(Nat.1, List.nil[Nat])))
                false
            }
            if not p5_c = p5_d {
                if p5_c = p5_e {
                    p5_c = List.cons(Nat.3, List.cons(Nat.2, List.nil[Nat]))
                    p5_e = List.cons(Nat.2, List.cons(Nat.2, List.cons(Nat.1, List.nil[Nat])))
                    List.cons(Nat.3, List.cons(Nat.2, List.nil[Nat])) = List.cons(Nat.2, List.cons(Nat.2, List.cons(Nat.1, List.nil[Nat])))
                    false
                }
                if not p5_c = p5_e {
                    if p5_c = p5_f {
                        p5_c = List.cons(Nat.3, List.cons(Nat.2, List.nil[Nat]))
                        p5_f = List.cons(Nat.2, List.cons(Nat.1, List.cons(Nat.1, List.cons(Nat.1, List.nil[Nat]))))
                        List.cons(Nat.3, List.cons(Nat.2, List.nil[Nat])) = List.cons(Nat.2, List.cons(Nat.1, List.cons(Nat.1, List.cons(Nat.1, List.nil[Nat]))))
                        false
                    }
                    if not p5_c = p5_f {
                        if p5_c = p5_g {
                            p5_c = List.cons(Nat.3, List.cons(Nat.2, List.nil[Nat]))
                            p5_g = List.cons(Nat.1, List.cons(Nat.1, List.cons(Nat.1, List.cons(Nat.1, List.cons(Nat.1, List.nil[Nat])))))
                            List.cons(Nat.3, List.cons(Nat.2, List.nil[Nat])) = List.cons(Nat.1, List.cons(Nat.1, List.cons(Nat.1, List.cons(Nat.1, List.cons(Nat.1, List.nil[Nat])))))
                            false
                        }
                    }
                    false
                }
                false
            }
            false
    }
    not List.cons(p5_d, List.cons(p5_e, List.cons(p5_f, List.cons(p5_g, List.nil[List[Nat]])))).contains(p5_c)
}

/// The partition p5_d does not occur among the later partitions.
theorem not_contains_p5_d {
    not List.cons(p5_e, List.cons(p5_f, List.cons(p5_g, List.nil[List[Nat]]))).contains(p5_d)
} by {
    if List.cons(p5_e, List.cons(p5_f, List.cons(p5_g, List.nil[List[Nat]]))).contains(p5_d) {
        cons_contains_eq(p5_e, List.cons(p5_f, List.cons(p5_g, List.nil[List[Nat]])), p5_d)
        List.cons(p5_e, List.cons(p5_f, List.cons(p5_g, List.nil[List[Nat]]))).contains(p5_d) =
            (p5_d = p5_e or List.cons(p5_f, List.cons(p5_g, List.nil[List[Nat]])).contains(p5_d))
        cons_contains_eq(p5_f, List.cons(p5_g, List.nil[List[Nat]]), p5_d)
        List.cons(p5_f, List.cons(p5_g, List.nil[List[Nat]])).contains(p5_d) =
            (p5_d = p5_f or List.cons(p5_g, List.nil[List[Nat]]).contains(p5_d))
        cons_contains_eq(p5_g, List.nil[List[Nat]], p5_d)
        List.cons(p5_g, List.nil[List[Nat]]).contains(p5_d) =
            (p5_d = p5_g or List.nil[List[Nat]].contains(p5_d))
        nil_not_contains(p5_d)
        p5_d = p5_e or p5_d = p5_f or p5_d = p5_g
            if p5_d = p5_e {
                p5_d = List.cons(Nat.3, List.cons(Nat.1, List.cons(Nat.1, List.nil[Nat])))
                p5_e = List.cons(Nat.2, List.cons(Nat.2, List.cons(Nat.1, List.nil[Nat])))
                List.cons(Nat.3, List.cons(Nat.1, List.cons(Nat.1, List.nil[Nat]))) = List.cons(Nat.2, List.cons(Nat.2, List.cons(Nat.1, List.nil[Nat])))
                false
            }
            if not p5_d = p5_e {
                if p5_d = p5_f {
                    p5_d = List.cons(Nat.3, List.cons(Nat.1, List.cons(Nat.1, List.nil[Nat])))
                    p5_f = List.cons(Nat.2, List.cons(Nat.1, List.cons(Nat.1, List.cons(Nat.1, List.nil[Nat]))))
                    List.cons(Nat.3, List.cons(Nat.1, List.cons(Nat.1, List.nil[Nat]))) = List.cons(Nat.2, List.cons(Nat.1, List.cons(Nat.1, List.cons(Nat.1, List.nil[Nat]))))
                    false
                }
                if not p5_d = p5_f {
                    if p5_d = p5_g {
                        p5_d = List.cons(Nat.3, List.cons(Nat.1, List.cons(Nat.1, List.nil[Nat])))
                        p5_g = List.cons(Nat.1, List.cons(Nat.1, List.cons(Nat.1, List.cons(Nat.1, List.cons(Nat.1, List.nil[Nat])))))
                        List.cons(Nat.3, List.cons(Nat.1, List.cons(Nat.1, List.nil[Nat]))) = List.cons(Nat.1, List.cons(Nat.1, List.cons(Nat.1, List.cons(Nat.1, List.cons(Nat.1, List.nil[Nat])))))
                        false
                    }
                }
                false
            }
            false
    }
    not List.cons(p5_e, List.cons(p5_f, List.cons(p5_g, List.nil[List[Nat]]))).contains(p5_d)
}

/// The partition p5_e does not occur among the later partitions.
theorem not_contains_p5_e {
    not List.cons(p5_f, List.cons(p5_g, List.nil[List[Nat]])).contains(p5_e)
} by {
    if List.cons(p5_f, List.cons(p5_g, List.nil[List[Nat]])).contains(p5_e) {
        cons_contains_eq(p5_f, List.cons(p5_g, List.nil[List[Nat]]), p5_e)
        List.cons(p5_f, List.cons(p5_g, List.nil[List[Nat]])).contains(p5_e) =
            (p5_e = p5_f or List.cons(p5_g, List.nil[List[Nat]]).contains(p5_e))
        cons_contains_eq(p5_g, List.nil[List[Nat]], p5_e)
        List.cons(p5_g, List.nil[List[Nat]]).contains(p5_e) =
            (p5_e = p5_g or List.nil[List[Nat]].contains(p5_e))
        nil_not_contains(p5_e)
        p5_e = p5_f or p5_e = p5_g
            if p5_e = p5_f {
                p5_e = List.cons(Nat.2, List.cons(Nat.2, List.cons(Nat.1, List.nil[Nat])))
                p5_f = List.cons(Nat.2, List.cons(Nat.1, List.cons(Nat.1, List.cons(Nat.1, List.nil[Nat]))))
                List.cons(Nat.2, List.cons(Nat.2, List.cons(Nat.1, List.nil[Nat]))) = List.cons(Nat.2, List.cons(Nat.1, List.cons(Nat.1, List.cons(Nat.1, List.nil[Nat]))))
                false
            }
            if not p5_e = p5_f {
                if p5_e = p5_g {
                    p5_e = List.cons(Nat.2, List.cons(Nat.2, List.cons(Nat.1, List.nil[Nat])))
                    p5_g = List.cons(Nat.1, List.cons(Nat.1, List.cons(Nat.1, List.cons(Nat.1, List.cons(Nat.1, List.nil[Nat])))))
                    List.cons(Nat.2, List.cons(Nat.2, List.cons(Nat.1, List.nil[Nat]))) = List.cons(Nat.1, List.cons(Nat.1, List.cons(Nat.1, List.cons(Nat.1, List.cons(Nat.1, List.nil[Nat])))))
                    false
                }
            }
            false
    }
    not List.cons(p5_f, List.cons(p5_g, List.nil[List[Nat]])).contains(p5_e)
}

/// The partition p5_f does not occur among the later partitions.
theorem not_contains_p5_f {
    not List.cons(p5_g, List.nil[List[Nat]]).contains(p5_f)
} by {
    if List.cons(p5_g, List.nil[List[Nat]]).contains(p5_f) {
        cons_contains_eq(p5_g, List.nil[List[Nat]], p5_f)
        List.cons(p5_g, List.nil[List[Nat]]).contains(p5_f) =
            (p5_f = p5_g or List.nil[List[Nat]].contains(p5_f))
        nil_not_contains(p5_f)
        p5_f = p5_g
            if p5_f = p5_g {
                p5_f = List.cons(Nat.2, List.cons(Nat.1, List.cons(Nat.1, List.cons(Nat.1, List.nil[Nat]))))
                p5_g = List.cons(Nat.1, List.cons(Nat.1, List.cons(Nat.1, List.cons(Nat.1, List.cons(Nat.1, List.nil[Nat])))))
                List.cons(Nat.2, List.cons(Nat.1, List.cons(Nat.1, List.cons(Nat.1, List.nil[Nat])))) = List.cons(Nat.1, List.cons(Nat.1, List.cons(Nat.1, List.cons(Nat.1, List.cons(Nat.1, List.nil[Nat])))))
                false
            }
    }
    not List.cons(p5_g, List.nil[List[Nat]]).contains(p5_f)
}

/// The singleton tail is unique.
theorem p5_suffix_unique_6 {
    List.cons(p5_g, List.nil[List[Nat]]).is_unique
} by {
    singleton_unique(p5_g)
    List.singleton[List[Nat]](p5_g).is_unique
    List.cons(p5_g, List.nil[List[Nat]]) = List.singleton[List[Nat]](p5_g)
    List.cons(p5_g, List.nil[List[Nat]]).is_unique
}

/// The tail [p5_f ... p5_g] is duplicate-free.
theorem p5_suffix_unique_5 {
    List.cons(p5_f, List.cons(p5_g, List.nil[List[Nat]])).is_unique
} by {
    p5_suffix_unique_6
    List.cons(p5_g, List.nil[List[Nat]]).is_unique
    not_contains_p5_f
    not List.cons(p5_g, List.nil[List[Nat]]).contains(p5_f)
    cons_unique_of_tail_unique_not_contains(p5_f, List.cons(p5_g, List.nil[List[Nat]]))
    List.cons(p5_g, List.nil[List[Nat]]).is_unique and not List.cons(p5_g, List.nil[List[Nat]]).contains(p5_f) implies List.cons(p5_f, List.cons(p5_g, List.nil[List[Nat]])).is_unique
    List.cons(p5_g, List.nil[List[Nat]]).is_unique and not List.cons(p5_g, List.nil[List[Nat]]).contains(p5_f)
    List.cons(p5_f, List.cons(p5_g, List.nil[List[Nat]])).is_unique
}

/// The tail [p5_e ... p5_g] is duplicate-free.
theorem p5_suffix_unique_4 {
    List.cons(p5_e, List.cons(p5_f, List.cons(p5_g, List.nil[List[Nat]]))).is_unique
} by {
    p5_suffix_unique_5
    List.cons(p5_f, List.cons(p5_g, List.nil[List[Nat]])).is_unique
    not_contains_p5_e
    not List.cons(p5_f, List.cons(p5_g, List.nil[List[Nat]])).contains(p5_e)
    cons_unique_of_tail_unique_not_contains(p5_e, List.cons(p5_f, List.cons(p5_g, List.nil[List[Nat]])))
    List.cons(p5_f, List.cons(p5_g, List.nil[List[Nat]])).is_unique and not List.cons(p5_f, List.cons(p5_g, List.nil[List[Nat]])).contains(p5_e) implies List.cons(p5_e, List.cons(p5_f, List.cons(p5_g, List.nil[List[Nat]]))).is_unique
    List.cons(p5_f, List.cons(p5_g, List.nil[List[Nat]])).is_unique and not List.cons(p5_f, List.cons(p5_g, List.nil[List[Nat]])).contains(p5_e)
    List.cons(p5_e, List.cons(p5_f, List.cons(p5_g, List.nil[List[Nat]]))).is_unique
}

/// The tail [p5_d ... p5_g] is duplicate-free.
theorem p5_suffix_unique_3 {
    List.cons(p5_d, List.cons(p5_e, List.cons(p5_f, List.cons(p5_g, List.nil[List[Nat]])))).is_unique
} by {
    p5_suffix_unique_4
    List.cons(p5_e, List.cons(p5_f, List.cons(p5_g, List.nil[List[Nat]]))).is_unique
    not_contains_p5_d
    not List.cons(p5_e, List.cons(p5_f, List.cons(p5_g, List.nil[List[Nat]]))).contains(p5_d)
    cons_unique_of_tail_unique_not_contains(p5_d, List.cons(p5_e, List.cons(p5_f, List.cons(p5_g, List.nil[List[Nat]]))))
    List.cons(p5_e, List.cons(p5_f, List.cons(p5_g, List.nil[List[Nat]]))).is_unique and not List.cons(p5_e, List.cons(p5_f, List.cons(p5_g, List.nil[List[Nat]]))).contains(p5_d) implies List.cons(p5_d, List.cons(p5_e, List.cons(p5_f, List.cons(p5_g, List.nil[List[Nat]])))).is_unique
    List.cons(p5_e, List.cons(p5_f, List.cons(p5_g, List.nil[List[Nat]]))).is_unique and not List.cons(p5_e, List.cons(p5_f, List.cons(p5_g, List.nil[List[Nat]]))).contains(p5_d)
    List.cons(p5_d, List.cons(p5_e, List.cons(p5_f, List.cons(p5_g, List.nil[List[Nat]])))).is_unique
}

/// The tail [p5_c ... p5_g] is duplicate-free.
theorem p5_suffix_unique_2 {
    List.cons(p5_c, List.cons(p5_d, List.cons(p5_e, List.cons(p5_f, List.cons(p5_g, List.nil[List[Nat]]))))).is_unique
} by {
    p5_suffix_unique_3
    List.cons(p5_d, List.cons(p5_e, List.cons(p5_f, List.cons(p5_g, List.nil[List[Nat]])))).is_unique
    not_contains_p5_c
    not List.cons(p5_d, List.cons(p5_e, List.cons(p5_f, List.cons(p5_g, List.nil[List[Nat]])))).contains(p5_c)
    cons_unique_of_tail_unique_not_contains(p5_c, List.cons(p5_d, List.cons(p5_e, List.cons(p5_f, List.cons(p5_g, List.nil[List[Nat]])))))
    List.cons(p5_d, List.cons(p5_e, List.cons(p5_f, List.cons(p5_g, List.nil[List[Nat]])))).is_unique and not List.cons(p5_d, List.cons(p5_e, List.cons(p5_f, List.cons(p5_g, List.nil[List[Nat]])))).contains(p5_c) implies List.cons(p5_c, List.cons(p5_d, List.cons(p5_e, List.cons(p5_f, List.cons(p5_g, List.nil[List[Nat]]))))).is_unique
    List.cons(p5_d, List.cons(p5_e, List.cons(p5_f, List.cons(p5_g, List.nil[List[Nat]])))).is_unique and not List.cons(p5_d, List.cons(p5_e, List.cons(p5_f, List.cons(p5_g, List.nil[List[Nat]])))).contains(p5_c)
    List.cons(p5_c, List.cons(p5_d, List.cons(p5_e, List.cons(p5_f, List.cons(p5_g, List.nil[List[Nat]]))))).is_unique
}

/// The tail [p5_b ... p5_g] is duplicate-free.
theorem p5_suffix_unique_1 {
    List.cons(p5_b, List.cons(p5_c, List.cons(p5_d, List.cons(p5_e, List.cons(p5_f, List.cons(p5_g, List.nil[List[Nat]])))))).is_unique
} by {
    p5_suffix_unique_2
    List.cons(p5_c, List.cons(p5_d, List.cons(p5_e, List.cons(p5_f, List.cons(p5_g, List.nil[List[Nat]]))))).is_unique
    not_contains_p5_b
    not List.cons(p5_c, List.cons(p5_d, List.cons(p5_e, List.cons(p5_f, List.cons(p5_g, List.nil[List[Nat]]))))).contains(p5_b)
    cons_unique_of_tail_unique_not_contains(p5_b, List.cons(p5_c, List.cons(p5_d, List.cons(p5_e, List.cons(p5_f, List.cons(p5_g, List.nil[List[Nat]]))))))
    List.cons(p5_c, List.cons(p5_d, List.cons(p5_e, List.cons(p5_f, List.cons(p5_g, List.nil[List[Nat]]))))).is_unique and not List.cons(p5_c, List.cons(p5_d, List.cons(p5_e, List.cons(p5_f, List.cons(p5_g, List.nil[List[Nat]]))))).contains(p5_b) implies List.cons(p5_b, List.cons(p5_c, List.cons(p5_d, List.cons(p5_e, List.cons(p5_f, List.cons(p5_g, List.nil[List[Nat]])))))).is_unique
    List.cons(p5_c, List.cons(p5_d, List.cons(p5_e, List.cons(p5_f, List.cons(p5_g, List.nil[List[Nat]]))))).is_unique and not List.cons(p5_c, List.cons(p5_d, List.cons(p5_e, List.cons(p5_f, List.cons(p5_g, List.nil[List[Nat]]))))).contains(p5_b)
    List.cons(p5_b, List.cons(p5_c, List.cons(p5_d, List.cons(p5_e, List.cons(p5_f, List.cons(p5_g, List.nil[List[Nat]])))))).is_unique
}

/// The tail [p5_a ... p5_g] is duplicate-free.
theorem p5_suffix_unique_0 {
    List.cons(p5_a, List.cons(p5_b, List.cons(p5_c, List.cons(p5_d, List.cons(p5_e, List.cons(p5_f, List.cons(p5_g, List.nil[List[Nat]]))))))).is_unique
} by {
    p5_suffix_unique_1
    List.cons(p5_b, List.cons(p5_c, List.cons(p5_d, List.cons(p5_e, List.cons(p5_f, List.cons(p5_g, List.nil[List[Nat]])))))).is_unique
    not_contains_p5_a
    not List.cons(p5_b, List.cons(p5_c, List.cons(p5_d, List.cons(p5_e, List.cons(p5_f, List.cons(p5_g, List.nil[List[Nat]])))))).contains(p5_a)
    cons_unique_of_tail_unique_not_contains(p5_a, List.cons(p5_b, List.cons(p5_c, List.cons(p5_d, List.cons(p5_e, List.cons(p5_f, List.cons(p5_g, List.nil[List[Nat]])))))))
    List.cons(p5_b, List.cons(p5_c, List.cons(p5_d, List.cons(p5_e, List.cons(p5_f, List.cons(p5_g, List.nil[List[Nat]])))))).is_unique and not List.cons(p5_b, List.cons(p5_c, List.cons(p5_d, List.cons(p5_e, List.cons(p5_f, List.cons(p5_g, List.nil[List[Nat]])))))).contains(p5_a) implies List.cons(p5_a, List.cons(p5_b, List.cons(p5_c, List.cons(p5_d, List.cons(p5_e, List.cons(p5_f, List.cons(p5_g, List.nil[List[Nat]]))))))).is_unique
    List.cons(p5_b, List.cons(p5_c, List.cons(p5_d, List.cons(p5_e, List.cons(p5_f, List.cons(p5_g, List.nil[List[Nat]])))))).is_unique and not List.cons(p5_b, List.cons(p5_c, List.cons(p5_d, List.cons(p5_e, List.cons(p5_f, List.cons(p5_g, List.nil[List[Nat]])))))).contains(p5_a)
    List.cons(p5_a, List.cons(p5_b, List.cons(p5_c, List.cons(p5_d, List.cons(p5_e, List.cons(p5_f, List.cons(p5_g, List.nil[List[Nat]]))))))).is_unique
}

/// The explicit list of the partitions of five is duplicate-free.
theorem partitions_five_unique {
    partitions_five.is_unique
} by {
    p5_suffix_unique_0
    List.cons(p5_a, List.cons(p5_b, List.cons(p5_c, List.cons(p5_d, List.cons(p5_e, List.cons(p5_f, List.cons(p5_g, List.nil[List[Nat]]))))))).is_unique
    partitions_five = List.cons(p5_a, List.cons(p5_b, List.cons(p5_c, List.cons(p5_d, List.cons(p5_e, List.cons(p5_f, List.cons(p5_g, List.nil[List[Nat]])))))))
    partitions_five.is_unique
}

/// The partitions of five are 5, 4 + 1, 3 + 2, 3 + 1 + 1, 2 + 2 + 1,
/// 2 + 1 + 1 + 1 and 1 + 1 + 1 + 1 + 1, so p(5) = 7.
theorem p_five {
    p(Nat.5) = Nat.7
} by {
    partition_set(Nat.5).cardinality_is(p(Nat.5))
    contains_set_intro(partitions_five, partition_set(Nat.5))
    forall(l: List[Nat]) {
        if partition_set(Nat.5).contains(l) {
            partition_set_contains_iff(Nat.5, l)
            partition_set(Nat.5).contains(l) = is_partition(l, Nat.5)
            is_partition(l, Nat.5)
            partitions_five_contains(l)
            is_partition(l, Nat.5) implies partitions_five.contains(l)
            partitions_five.contains(l)
        }
    }
    forall(l: List[Nat]) { partition_set(Nat.5).contains(l) implies partitions_five.contains(l) }
    partitions_five.contains_set(partition_set(Nat.5))
    partition_five_char(p5_a)
    is_partition(p5_a, Nat.5) = (p5_a = p5_a or p5_a = p5_b or p5_a = p5_c or p5_a = p5_d or p5_a = p5_e or p5_a = p5_f or p5_a = p5_g)
    p5_a = p5_a
    is_partition(p5_a, Nat.5) = true
    partition_set_contains_iff(Nat.5, p5_a)
    partition_set(Nat.5).contains(p5_a) = is_partition(p5_a, Nat.5)
    partition_set(Nat.5).contains(p5_a) = true
    partition_set(Nat.5).contains(p5_a)
    partition_five_char(p5_b)
    is_partition(p5_b, Nat.5) = (p5_b = p5_a or p5_b = p5_b or p5_b = p5_c or p5_b = p5_d or p5_b = p5_e or p5_b = p5_f or p5_b = p5_g)
    p5_b = p5_b
    is_partition(p5_b, Nat.5) = true
    partition_set_contains_iff(Nat.5, p5_b)
    partition_set(Nat.5).contains(p5_b) = true
    partition_set(Nat.5).contains(p5_b)
    partition_five_char(p5_c)
    is_partition(p5_c, Nat.5) = (p5_c = p5_a or p5_c = p5_b or p5_c = p5_c or p5_c = p5_d or p5_c = p5_e or p5_c = p5_f or p5_c = p5_g)
    p5_c = p5_c
    is_partition(p5_c, Nat.5) = true
    partition_set_contains_iff(Nat.5, p5_c)
    partition_set(Nat.5).contains(p5_c) = true
    partition_set(Nat.5).contains(p5_c)
    partition_five_char(p5_d)
    is_partition(p5_d, Nat.5) = (p5_d = p5_a or p5_d = p5_b or p5_d = p5_c or p5_d = p5_d or p5_d = p5_e or p5_d = p5_f or p5_d = p5_g)
    p5_d = p5_d
    is_partition(p5_d, Nat.5) = true
    partition_set_contains_iff(Nat.5, p5_d)
    partition_set(Nat.5).contains(p5_d) = true
    partition_set(Nat.5).contains(p5_d)
    partition_five_char(p5_e)
    is_partition(p5_e, Nat.5) = (p5_e = p5_a or p5_e = p5_b or p5_e = p5_c or p5_e = p5_d or p5_e = p5_e or p5_e = p5_f or p5_e = p5_g)
    p5_e = p5_e
    is_partition(p5_e, Nat.5) = true
    partition_set_contains_iff(Nat.5, p5_e)
    partition_set(Nat.5).contains(p5_e) = true
    partition_set(Nat.5).contains(p5_e)
    partition_five_char(p5_f)
    is_partition(p5_f, Nat.5) = (p5_f = p5_a or p5_f = p5_b or p5_f = p5_c or p5_f = p5_d or p5_f = p5_e or p5_f = p5_f or p5_f = p5_g)
    p5_f = p5_f
    is_partition(p5_f, Nat.5) = true
    partition_set_contains_iff(Nat.5, p5_f)
    partition_set(Nat.5).contains(p5_f) = true
    partition_set(Nat.5).contains(p5_f)
    partition_five_char(p5_g)
    is_partition(p5_g, Nat.5) = (p5_g = p5_a or p5_g = p5_b or p5_g = p5_c or p5_g = p5_d or p5_g = p5_e or p5_g = p5_f or p5_g = p5_g)
    p5_g = p5_g
    is_partition(p5_g, Nat.5) = true
    partition_set_contains_iff(Nat.5, p5_g)
    partition_set(Nat.5).contains(p5_g) = true
    partition_set(Nat.5).contains(p5_g)
    forall(x: List[Nat]) {
        if partitions_five.contains(x) {
            partitions_five_membership(x)
            partitions_five.contains(x) implies is_partition(x, Nat.5)
            is_partition(x, Nat.5)
            partition_set_contains_iff(Nat.5, x)
            partition_set(Nat.5).contains(x) = is_partition(x, Nat.5)
            partition_set(Nat.5).contains(x)
        }
    }
    forall(x: List[Nat]) { partitions_five.contains(x) implies partition_set(Nat.5).contains(x) }
    filter_all_keep(partitions_five, partition_set(Nat.5).contains)
    (forall(x: List[Nat]) { partitions_five.contains(x) implies partition_set(Nat.5).contains(x) }) implies partitions_five.filter(partition_set(Nat.5).contains) = partitions_five
    partitions_five.filter(partition_set(Nat.5).contains) = partitions_five
    partitions_five_unique
    partitions_five.is_unique
    partitions_five.unique = partitions_five
    partitions_five.length = List.cons(p5_b, List.cons(p5_c, List.cons(p5_d, List.cons(p5_e, List.cons(p5_f, List.cons(p5_g, List.nil[List[Nat]])))))).length.suc
    List.cons(p5_b, List.cons(p5_c, List.cons(p5_d, List.cons(p5_e, List.cons(p5_f, List.cons(p5_g, List.nil[List[Nat]])))))).length = List.cons(p5_c, List.cons(p5_d, List.cons(p5_e, List.cons(p5_f, List.cons(p5_g, List.nil[List[Nat]]))))).length.suc
    List.cons(p5_c, List.cons(p5_d, List.cons(p5_e, List.cons(p5_f, List.cons(p5_g, List.nil[List[Nat]]))))).length = List.cons(p5_d, List.cons(p5_e, List.cons(p5_f, List.cons(p5_g, List.nil[List[Nat]])))).length.suc
    List.cons(p5_d, List.cons(p5_e, List.cons(p5_f, List.cons(p5_g, List.nil[List[Nat]])))).length = List.cons(p5_e, List.cons(p5_f, List.cons(p5_g, List.nil[List[Nat]]))).length.suc
    List.cons(p5_e, List.cons(p5_f, List.cons(p5_g, List.nil[List[Nat]]))).length = List.cons(p5_f, List.cons(p5_g, List.nil[List[Nat]])).length.suc
    List.cons(p5_f, List.cons(p5_g, List.nil[List[Nat]])).length = List.cons(p5_g, List.nil[List[Nat]]).length.suc
    List.cons(p5_g, List.nil[List[Nat]]).length = List.nil[List[Nat]].length.suc
    List.nil[List[Nat]].length = Nat.0
    List.cons(p5_g, List.nil[List[Nat]]).length = Nat.0.suc
    List.cons(p5_f, List.cons(p5_g, List.nil[List[Nat]])).length = Nat.0.suc.suc
    List.cons(p5_e, List.cons(p5_f, List.cons(p5_g, List.nil[List[Nat]]))).length = Nat.0.suc.suc.suc
    List.cons(p5_d, List.cons(p5_e, List.cons(p5_f, List.cons(p5_g, List.nil[List[Nat]])))).length = Nat.0.suc.suc.suc.suc
    List.cons(p5_c, List.cons(p5_d, List.cons(p5_e, List.cons(p5_f, List.cons(p5_g, List.nil[List[Nat]]))))).length = Nat.0.suc.suc.suc.suc.suc
    List.cons(p5_b, List.cons(p5_c, List.cons(p5_d, List.cons(p5_e, List.cons(p5_f, List.cons(p5_g, List.nil[List[Nat]])))))).length = Nat.0.suc.suc.suc.suc.suc.suc
    partitions_five.length = Nat.0.suc.suc.suc.suc.suc.suc.suc
    Nat.0.suc.suc.suc.suc.suc.suc.suc = Nat.7
    partitions_five.length = Nat.7
    partitions_five.filter(partition_set(Nat.5).contains).unique = partitions_five
    partitions_five.filter(partition_set(Nat.5).contains).unique.length = Nat.7
    partitions_five.contains_set(partition_set(Nat.5)) and
        partitions_five.filter(partition_set(Nat.5).contains).unique.length = Nat.7
    exists(containing_list: List[List[Nat]]) {
        containing_list.contains_set(partition_set(Nat.5)) and
            containing_list.filter(partition_set(Nat.5).contains).unique.length = Nat.7
    }
    partition_set(Nat.5).cardinality_is(Nat.7)
    cardinality_is_well_defined(partition_set(Nat.5), p(Nat.5), Nat.7)
    partition_set(Nat.5).cardinality_is(p(Nat.5)) and partition_set(Nat.5).cardinality_is(Nat.7) implies p(Nat.5) = Nat.7
    p(Nat.5) = Nat.7
}
