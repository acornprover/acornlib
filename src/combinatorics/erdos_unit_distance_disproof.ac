/// The 2026 disproof of the Erdős unit-distance conjecture: foundations.
///
/// Alon–Bloom–Gowers–Litt–Sawin–Shankar–Tsimerman–Wang–Wood (arXiv
/// 2605.20695) disproved the Erdős unit-distance conjecture: for infinitely
/// many n there are planar point sets of size n with more than n^(1 + eps)
/// unit distances, for a fixed eps > 0.  The construction lives in the
/// rational plane and is driven by quaternion-algebra norms (following
/// Golod–Shafarevich, Ellenberg–Venkatesh and Hajir–Maire–Ramakrishna):
/// a set of points is built in which many pairwise differences have norm
/// one, and the norm-one relations come from the multiplicative structure
/// of the quaternion norm.
///
/// The algebraic engine that this library formalizes is in
/// `number_theory/four_squares_identity.ac` and
/// `number_theory/quaternion_unit_distance.ac`:
///
///   * Euler's four-squares identity — the quaternion norm
///     N(a, b, c, d) = a² + b² + c² + d² is multiplicative, proved in the
///     Cayley–Dickson presentation over pairs of complex numbers
///     (`norm2_mul`, `four_squares_identity`), and
///   * the unit-distance interpretation — identifying R⁴ with the
///     quaternions, the squared Euclidean distance of two points is the
///     norm of their difference (`quat_dist_sq_eq_norm_sub`), and right
///     multiplication by a unit quaternion is an isometry
///     (`quat_unit_dist_preserved`), so it maps unit distances to unit
///     distances (`quat_unit_dist_iff`).
///
/// The counting machinery of the disproof is PROVED in
/// `number_theory/quaternion_unit_distance.ac`: for a finite set `s` of
/// quaternions, `nu(s)` counts the unordered unit pairs, and
///
///   nu_unit_invariant:  N(q) = 1 implies nu(s·q) = nu(s)
///
/// (right multiplication by a unit quaternion is a bijection — its inverse
/// is right multiplication by the conjugate — that preserves exactly the
/// unit pairs).  Toward the counting of unit pairs in the four-dimensional
/// integer lattice, `number_theory/quaternion_grid_count.ac` proves the
/// coordinate bounds: each coordinate of a natural solution of
/// a² + b² + c² + d² = 1 is at most one (`from_nat_sq_le_one_imp_le_one`),
/// hence zero or one (`nat_le_one_zero_or_one`), and any two ones make the
/// sum exceed one (`four_sq_two_ones_gt_one` and its five permutations).
/// The full characterization — exactly one coordinate equals one
/// (`four_sq_eq_one`) — and the grid count it feeds (an n by n by n by n
/// grid of naturals has 4·n³·(n−1) unordered unit pairs) are recorded as
/// commented statements in that file.
///
/// What remains for the full disproof is the set construction: one needs a
/// finite set `s` for which many pairwise quotients have norm one, forcing
/// `nu(s)` to be large.  The Golod–Shafarevich group construction behind
/// the 2026 result is recorded as the commented theorem text below.
from nat import Nat
from real import Real

numerals Real
numerals Nat

/// Four real numbers package the coordinates of a point of R⁴.
structure Point4 {
    /// The first coordinate.
    a: Real
    /// The second coordinate.
    b: Real
    /// The third coordinate.
    c: Real
    /// The fourth coordinate.
    d: Real
}

/// The squared distance of two points of R⁴.
define dist_sq(p: Point4, q: Point4) -> Real {
    (p.a - q.a) * (p.a - q.a) + (p.b - q.b) * (p.b - q.b) +
    (p.c - q.c) * (p.c - q.c) + (p.d - q.d) * (p.d - q.d)
}

/// True when two points of R⁴ are at Euclidean distance exactly one.
define points_unit_distance4(p: Point4, q: Point4) -> Bool {
    dist_sq(p, q) = Real.1
}

// The remaining statement of the disproof.  The invariance
// `nu_unit_invariant` (proved in number_theory/quaternion_unit_distance.ac)
// lets one replace a set by any unit-quaternion translate; the disproof
// then needs a set s for which many pairwise quotients have norm one.  The
// Golod-Shafarevich construction is not yet formalized:
//
//   theorem erdos_unit_distance_disproved_2026 {
//       exists(c: Real) {
//           Real.1 < c and forall(n0: Nat) {
//               exists(n: Nat) {
//                   n0 <= n and exists(s: FiniteSet[Point4]) {
//                       s.cardinality = n and c * n <= nu(s)
//                   }
//               }
//           }
//       }
//   }
//
// The two-dimensional grid bound proved in `erdos_unit_distance.ac`
// (u(n²) >= 2·n·(n - 1)) is the classical lower-bound side; the 2026
// construction exceeds every polynomial of exponent one.
