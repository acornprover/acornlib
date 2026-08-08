/// Public interface for combinatorics.

// binomial.ac
from nat import Nat
from nat import nat_mul_3_2
from nat import pow_one
from nat import alt_induction, distrib_left, distrib_right
from list import partial
from list import partial_one, partial_split_last, partial_drop_first, partial_add,
    partial_pointwise_eq
from algebra.semigroup import mul_fn
from algebra.add_semigroup import add_fn
from data.basic.functions import compose

numerals Nat

define choose_exists(x: Nat, y: Nat) -> Bool {
    (x.factorial * y.factorial).divides((x + y).factorial)
}

define choose_hyp(n: Nat) -> Bool {
    forall(x: Nat, y: Nat) {
        if x + y = n {
            choose_exists(x, y)
        }
    }
}

theorem choose_exists_left_zero(x: Nat) {
    choose_exists(0, x)
}

theorem choose_exists_right_zero(x: Nat) {
    choose_exists(x, 0)
}

theorem choose_base {
    choose_hyp(0)
}

theorem choose_lemma(a: Nat, b: Nat) {
    a != 0 and b != 0 and choose_hyp(a + b - 1) implies
    (a.factorial * b.factorial).divides((a + b - 1).factorial * b)
}

theorem choose_exists_is_true(z1: Nat, z2: Nat) {
    choose_exists(z1, z2)
}

theorem choose_exists_sub_form(n: Nat, k: Nat) {
    k <= n implies exists(c: Nat) {
        c * k.factorial * (n - k).factorial = n.factorial
    }
}

/// The binomial coefficient "n choose k".
let binom(n: Nat, k: Nat) -> c: Nat satisfy {
    if n < k {
        c = 0
    } else {
        c * k.factorial * (n - k).factorial = n.factorial
    }
}

attributes Nat {
    /// The binomial coefficient "n choose k".
    let binom = binom
}

theorem factorial_nonzero(n: Nat) {
    n.factorial != 0
}

theorem choose_zero(n: Nat) {
    n.binom(0) = 1
}

theorem choose_n(n: Nat) {
    n.binom(n) = 1
}

theorem choose_add(a: Nat, b: Nat) {
    (a + b).binom(a) * a.factorial * b.factorial = (a + b).factorial
}

theorem choose_symm_add_form(a: Nat, b: Nat) {
    (a + b).binom(a) = (a + b).binom(b)
}

theorem choose_symm_sub_form(n: Nat, k: Nat) {
    k <= n implies n.binom(k) = n.binom(n - k)
}

theorem choose_one(n: Nat) {
    n.binom(1) = n
}

/// The number of ways to choose n - 1 items from n is n.
theorem binom_n_minus_one(n: Nat) {
    1 <= n implies n.binom(n - 1) = n
}

/// Twice "n choose 2" equals n times n minus one, when n is at least two.
theorem binom_two_double(n: Nat) {
    2 <= n implies 2 * n.binom(2) = n * (n - 1)
}

/// Six times "n choose 3" equals n times n minus one times n minus two, when n is at least three.
theorem binom_three_factor(n: Nat) {
    3 <= n implies 6 * n.binom(3) = n * (n - 1) * (n - 2)
}

/// When k > n, the binomial coefficient is zero.
theorem choose_out_of_bounds(n: Nat, k: Nat) {
    n < k implies n.binom(k) = 0
}

/// The binomial coefficient n choose k is positive whenever k is at most n.
theorem binom_pos(n: Nat, k: Nat) {
    k <= n implies 0 < n.binom(k)
}

/// Pascal's identity: each entry in Pascal's triangle is the sum of the two entries above it.
theorem pascal(n: Nat, k: Nat) {
    0 < k and k <= n implies n.suc.binom(k) = n.binom(k - 1) + n.binom(k)
}

/// Pascal's identity in successor form, free of subtraction.
theorem pascal_suc(n: Nat, k: Nat) {
    k <= n implies n.suc.binom(k.suc) = n.binom(k) + n.binom(k.suc)
}

define binomial_term(a: Nat, b: Nat, n: Nat, k: Nat) -> Nat {
    n.binom(k) * a.pow(k) * b.pow(n - k)
}

/// The boundary term at k=0 for the binomial expansion.
theorem binomial_term_zero(a: Nat, b: Nat, m: Nat) {
    binomial_term(a, b, m.suc, 0) = b * binomial_term(a, b, m, 0)
}

/// The boundary term at k=m+1 for the binomial expansion.
theorem binomial_term_top(a: Nat, b: Nat, m: Nat) {
    binomial_term(a, b, m.suc, m.suc) = a * binomial_term(a, b, m, m)
}

/// Recursive relation for binomial terms: each term for n+1 equals a times the shifted
/// term for n plus b times the corresponding term for n.
theorem binomial_term_recurrence(a: Nat, b: Nat, m: Nat, k: Nat) {
    0 < k and k <= m implies
    binomial_term(a, b, m.suc, k) =
    a * binomial_term(a, b, m, k - 1) + b * binomial_term(a, b, m, k)
}

/// Alternative form of the binomial term recurrence, avoiding subtraction by using k.suc.
theorem alt_binomial_term_recurrence(a: Nat, b: Nat, m: Nat, k: Nat) {
    k < m implies
    binomial_term(a, b, m.suc, k.suc) =
    a * binomial_term(a, b, m, k) + b * binomial_term(a, b, m, k.suc)
}

/// Helper: the middle partial sum splits into two parts using the recurrence.
theorem binomial_middle_sum(a: Nat, b: Nat, m: Nat) {
    partial(compose(binomial_term(a, b, m.suc), Nat.suc), m) =
        a * partial(binomial_term(a, b, m), m) + b * partial(compose(binomial_term(a, b, m), Nat.suc), m)
}

/// Distributing (a + b) across a partial sum of binomial terms gives the next row.
theorem binomial_distribution(a: Nat, b: Nat, m: Nat) {
    (a + b) * partial(binomial_term(a, b, m), m.suc) = partial(binomial_term(a, b, m.suc), m.suc.suc)
}

/// The binomial theorem: (a + b)^n equals the sum of binomial terms.
theorem binomial(a: Nat, b: Nat, n: Nat) {
    (a + b).pow(n) = partial(binomial_term(a, b, n), n.suc)
}

/// Absorption identity: k * (n choose k) = n * (n - 1 choose k - 1) when 0 < k <= n.
theorem binom_absorption(n: Nat, k: Nat) {
    0 < k and k <= n implies k * n.binom(k) = n * (n - 1).binom(k - 1)
}

/// Absorption identity in successor form, free of subtraction.
theorem binom_absorption_suc(n: Nat, k: Nat) {
    k <= n implies k.suc * n.suc.binom(k.suc) = n.suc * n.binom(k)
}

/// Complementary absorption identity: (n - k) * binom(n, k) = (k + 1) * binom(n, k + 1) when k < n.
theorem binom_complement_absorption(n: Nat, k: Nat) {
    k < n implies (n - k) * n.binom(k) = k.suc * n.binom(k.suc)
}

/// Binomial coefficients are weakly increasing in the left half: when 2 * (k + 1) is
/// at most n, n choose k is at most n choose k + 1.
theorem binom_le_binom_suc(n: Nat, k: Nat) {
    2 * k.suc <= n implies n.binom(k) <= n.binom(k.suc)
}

/// Binomial coefficients are weakly decreasing in the right half: when n is at most 2 * k,
/// n choose k + 1 is at most n choose k.
theorem binom_suc_le_binom(n: Nat, k: Nat) {
    n <= 2 * k implies n.binom(k.suc) <= n.binom(k)
}

/// Subset selection identity: choosing a k-subset and then a j-subset of it is the same
/// as choosing the j-subset first and then completing it to a k-subset.
theorem binom_subset_product(n: Nat, k: Nat, j: Nat) {
    j <= k and k <= n implies n.binom(k) * k.binom(j) = n.binom(j) * (n - j).binom(k - j)
}

/// Pascal's identity in successor form, valid for all k and n without any bound.
theorem pascal_suc_unbounded(n: Nat, k: Nat) {
    n.suc.binom(k.suc) = n.binom(k) + n.binom(k.suc)
}

/// The j-th term in Vandermonde's convolution for choosing k items from m + n.
define vandermonde_term(m: Nat, n: Nat, k: Nat, j: Nat) -> Nat {
    m.binom(j) * n.binom(k - j)
}

/// Auxiliary lemma: the partial sum of vandermonde_term at m = 0 equals n.binom(k)
/// for any positive upper bound, since only the j = 0 term is nonzero.
theorem vandermonde_zero_aux(n: Nat, k: Nat, m: Nat) {
    partial(vandermonde_term(Nat.0, n, k), m.suc) = n.binom(k)
}

/// Vandermonde's identity, inductive step: assuming Vandermonde holds for big_m at every k,
/// it holds for big_m.suc at every k1.
theorem vandermonde_step(big_m: Nat, n: Nat, k1: Nat) {
    (forall(kk: Nat) {
        partial(vandermonde_term(big_m, n, kk), kk.suc) = (big_m + n).binom(kk)
    }) implies partial(vandermonde_term(big_m.suc, n, k1), k1.suc) = (big_m.suc + n).binom(k1)
}

/// Vandermonde's identity for binomial coefficients.
theorem vandermonde(m: Nat, n: Nat, k: Nat) {
    partial(vandermonde_term(m, n, k), k.suc) = (m + n).binom(k)
}

/// The square of the `j`th binomial coefficient in row `n`.
define binom_square_term(n: Nat, j: Nat) -> Nat {
    n.binom(j) * n.binom(j)
}

/// The sum of the squares of the binomial coefficients in row `n` is the
/// central binomial coefficient in row `2*n`.
theorem binom_square_sum(n: Nat) {
    partial(binom_square_term(n), n.suc) = (n + n).binom(n)
}

/// The `j`th term in the hockey-stick sum with fixed lower index `r`.
define hockey_stick_term(r: Nat, j: Nat) -> Nat {
    (r + j).binom(r)
}

/// The hockey-stick identity for binomial coefficients.
theorem hockey_stick(r: Nat, n: Nat) {
    partial(hockey_stick_term(r), n.suc) = (r + n.suc).binom(r.suc)
}

/// The sum of a row of binomial coefficients equals 2 to the n.
theorem binom_row_sum(n: Nat) {
    partial(n.binom, n.suc) = 2.pow(n)
}

/// The `k`th coefficient in the first moment of binomial row `n`.
define binom_weighted_term(n: Nat, k: Nat) -> Nat {
    k * n.binom(k)
}

/// The first moment of binomial row `n` is `n * 2^(n - 1)`.
theorem binom_weighted_row_sum(n: Nat) {
    partial(binom_weighted_term(n), n.suc) = n * 2.pow(n - 1)
}

/// The `k`th coefficient in the second falling-factorial moment of binomial row `n`.
define binom_second_weighted_term(n: Nat, k: Nat) -> Nat {
    k * (k - Nat.1) * n.binom(k)
}

/// The second falling-factorial moment of binomial row `n` is
/// `n * (n - 1) * 2^(n - 2)`.
theorem binom_second_weighted_row_sum(n: Nat) {
    partial(binom_second_weighted_term(n), n.suc) =
        n * (n - Nat.1) * 2.pow((n - Nat.1) - Nat.1)
}

/// The `n`th triangular number: `0 + 1 + ... + n`.
define triangular(n: Nat) -> Nat {
    match n {
        Nat.zero {
            Nat.0
        }
        Nat.suc(k) {
            n + triangular(k)
        }
    }
}

/// Empty base case of `triangular`.
theorem triangular_zero {
    triangular(Nat.0) = Nat.0
}

/// Successor recurrence for `triangular`.
theorem triangular_suc(k: Nat) {
    triangular(k.suc) = k.suc + triangular(k)
}

/// Inductive step for `triangular_doubled`.
theorem triangular_doubled_suc(k: Nat) {
    Nat.2 * triangular(k) = k * (k + Nat.1)
        implies Nat.2 * triangular(k.suc) = k.suc * (k.suc + Nat.1)
}

/// `2 * triangular(n) = n * (n + 1)`. The closed form, in `Nat` (no division).
theorem triangular_doubled(n: Nat) {
    Nat.2 * triangular(n) = n * (n + Nat.1)
}
