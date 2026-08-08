from nat import Nat
from nat import nat_mul_3_2
from nat import alt_induction, distrib_left, distrib_right
from list import partial
from list import partial_one, partial_split_last, partial_drop_first, partial_add,
    partial_pointwise_eq, partial_scalar_mul
from algebra.semigroup import mul_fn
from algebra.add_semigroup import add_fn
from data.basic.functions import compose

numerals Nat

// This file contains basic combinatorics.

// We want to define "n choose k", but it's a bit tricky to prove it's a natural number.
// We're going to do induction, but it'll be simpler to separate these out into named functions.
// We are essentially proving that n choose k is integral using the Pascal's triangle recurrence,
// except we are expressing the steps without using rational numbers.

define choose_exists(x: Nat, y: Nat) -> Bool {
    (x.factorial * y.factorial).divides((x + y).factorial)
}

// This will be the inductive hypothesis
define choose_hyp(n: Nat) -> Bool {
    forall(x: Nat, y: Nat) {
        if x + y = n {
            choose_exists(x, y)
        }
    }
}

theorem choose_exists_left_zero(x: Nat) {
    choose_exists(0, x)
}

theorem choose_exists_right_zero(x: Nat) {
    choose_exists(x, 0)
}

theorem choose_base {
    choose_hyp(0)
} by {
    forall(x: Nat, y: Nat) {
        if x + y = Nat.0 {
            y = Nat.0
            choose_exists(x, y)
        }
    }
}

theorem choose_lemma(a: Nat, b: Nat) {
    a != 0 and b != 0 and choose_hyp(a + b - 1) implies
    (a.factorial * b.factorial).divides((a + b - 1).factorial * b)
} by {

    // a! (b - 1)! divides (a + b - 1)!
    not b < 0
    b - 1 + 1 = b or b < 1
    not b < 0.suc or b < 0 or 0 = b
    a + (b - 1).suc = (a + (b - 1)).suc
    (b - 1) + 1 = (b - 1).suc
    (a + (b - 1)).suc - 1 = a + (b - 1)
    a + b - 1 = a + (b - 1)

    // a! b! (b - 1)! divides (a + b - 1)! b!
    (a.factorial * (b - 1).factorial * b.factorial).divides((a + b - 1).factorial * b.factorial)

    // a! b! (b - 1)! divides (a + b - 1)! b * (b - 1)!
    b.factorial = b * (b - 1).factorial
    (a.factorial * b.factorial * (b - 1).factorial).divides((a + b - 1).factorial * b * (b - 1).factorial)

    // a! b! divides (a + b - 1)! b
}

theorem choose_exists_is_true(z1: Nat, z2: Nat) {
    choose_exists(z1, z2)
} by {
    forall(x: Nat) {
        if choose_hyp(x) {
            forall(a: Nat, b: Nat) {
                if a + b = x.suc {
                    if b = 0 {
                        choose_exists(a, b)
                    }
                    if a != 0 and b != 0 {
                        // Use the lemma one way
                        (a.factorial * b.factorial).divides((a + b - 1).factorial * b)

                        // Use the lemma the other way
                        (b.factorial * a.factorial).divides((b + a - 1).factorial * a)

                        (a + b - 1).factorial * b + (a + b - 1).factorial * a = (a + b).factorial
                        (a.factorial * b.factorial).divides((a + b).factorial)
                    }
                    choose_exists(a, b)
                }
            }
            choose_hyp(x.suc)
        }
    }

    choose_hyp(z1 + z2)
    z2 + z1 = z1 + z2
}

theorem choose_exists_sub_form(n: Nat, k: Nat) {
    k <= n implies exists(c: Nat) {
        c * k.factorial * (n - k).factorial = n.factorial
    }
} by {
    if k <= n {
        n - k + k = n
        ((n - k).factorial * k.factorial).divides(n.factorial)
        let c: Nat satisfy {
            ((n - k).factorial * k.factorial) * c = n.factorial
        }
        c * k.factorial * (n - k).factorial = n.factorial
        exists(c0: Nat) {
            c0 * k.factorial * (n - k).factorial = n.factorial
        }
    }
}

/// The binomial coefficient "n choose k".
let binom(n: Nat, k: Nat) -> c: Nat satisfy {
    if n < k {
        c = 0
    } else {
        c * k.factorial * (n - k).factorial = n.factorial
    }
} by {
    if n < k {
        let x: Nat satisfy {
            x = 0
        }
    } else {
        let x: Nat satisfy {
            x * k.factorial * (n - k).factorial = n.factorial
        }
    }
}

attributes Nat {
    /// The binomial coefficient "n choose k".
    let binom = binom
}

theorem factorial_nonzero(n: Nat) {
    n.factorial != 0
} by {
    1 <= n.factorial
    n.factorial + 1 = n.factorial.suc
}

theorem choose_zero(n: Nat) {
    n.binom(0) = 1
} by {
    n.binom(0) * 0.factorial * (n - 0).factorial = n.factorial
}

theorem choose_n(n: Nat) {
    n.binom(n) = 1
} by {
    n.binom(n) * n.factorial * (n - n).factorial = n.factorial
}

theorem choose_add(a: Nat, b: Nat) {
    (a + b).binom(a) * a.factorial * b.factorial = (a + b).factorial
} by {
    (a + b - a).factorial = b.factorial
    a <= a + b
    not a + b < a
}

theorem choose_symm_add_form(a: Nat, b: Nat) {
    (a + b).binom(a) = (a + b).binom(b)
} by {
    (a + b).binom(b) * (a.factorial * b.factorial) = (a + b).factorial
    (a + b).binom(a) * a.factorial * b.factorial = (a + b).factorial
    a.factorial != 0
    b.factorial != 0
    (a + b).binom(b) * (a.factorial * b.factorial) = (a + b).binom(b) * a.factorial * b.factorial
    (a + b).binom(b) * a.factorial * b.factorial = (a + b).binom(a) * a.factorial * b.factorial
    (a + b).binom(b) * a.factorial = (a + b).binom(a) * a.factorial
}

theorem choose_symm_sub_form(n: Nat, k: Nat) {
    k <= n implies n.binom(k) = n.binom(n - k)
} by {
    let a: Nat satisfy {
        k + a = n
    }
    k + a = a + k
    n - k = a
}

theorem choose_one(n: Nat) {
    n.binom(1) = n
} by {
    if n < 1 {
    } else {
        n.factorial = n * (n - 1).factorial
        n.binom(1) * 1.factorial * (n - 1).factorial = n.factorial
        n.binom(1) * 1 = n.binom(1)
        (n - 1).factorial != 0
        n.binom(1) = n
    }
}

/// The number of ways to choose n - 1 items from n is n.
theorem binom_n_minus_one(n: Nat) {
    1 <= n implies n.binom(n - 1) = n
} by {
    if 1 <= n {
        let a: Nat satisfy { 1 + a = n }
        n - 1 <= n
        n.binom(n - 1) = n.binom(1)
    }
}

/// Twice "n choose 2" equals n times n minus one, when n is at least two.
theorem binom_two_double(n: Nat) {
    2 <= n implies 2 * n.binom(2) = n * (n - 1)
} by {
    if 2 <= n {
        not n < 2
        n.binom(2) * 2.factorial * (n - 2).factorial = n.factorial
        2.factorial = 2
        n.binom(2) * 2 * (n - 2).factorial = n.factorial

        // Expand n.factorial = n * (n - 1) * (n - 2).factorial
        let c: Nat satisfy { 2 + c = n }
        c = n - 2
        2 + c = (1 + c).suc
        1 + c.suc = n
        c.suc = n - 1
        (n - 1).factorial = c.suc.factorial
        c.suc.factorial = c.suc * c.factorial
        (n - 1).factorial = (n - 1) * (n - 2).factorial
        n.factorial = n * (n - 1).factorial

        n.binom(2) * 2 * (n - 2).factorial = n * (n - 1) * (n - 2).factorial
        (n - 2).factorial != 0
        2 * n.binom(2) = n * (n - 1)
    }
}

/// Six times "n choose 3" equals n times n minus one times n minus two, when n is at least three.
theorem binom_three_factor(n: Nat) {
    3 <= n implies 6 * n.binom(3) = n * (n - 1) * (n - 2)
} by {
    if 3 <= n {
        not n < 3
        n.binom(3) * 3.factorial * (n - 3).factorial = n.factorial
        3.factorial = 3 * 2.factorial
        2.factorial = 2
        3.factorial = 3 * 2
        nat_mul_3_2
        3 * 2 = 6
        3.factorial = 6
        n.binom(3) * 6 * (n - 3).factorial = n.factorial

        // Unfold n.factorial = n * (n - 1) * (n - 2) * (n - 3).factorial
        let c: Nat satisfy { 3 + c = n }
        c = n - 3
        3 + c = (2 + c).suc
        2 + c.suc = n
        c.suc = n - 2
        2 + c = (1 + c).suc
        1 + c.suc.suc = n
        c.suc.suc = n - 1
        c.suc.factorial = c.suc * c.factorial
        c.suc.suc.factorial = c.suc.suc * c.suc.factorial
        (n - 2).factorial = (n - 2) * (n - 3).factorial
        (n - 1).factorial = (n - 1) * (n - 2).factorial
        (n - 1).factorial = (n - 1) * ((n - 2) * (n - 3).factorial)
        (n - 1).factorial = (n - 1) * (n - 2) * (n - 3).factorial
        n.factorial = n * (n - 1).factorial
        n.factorial = n * ((n - 1) * (n - 2) * (n - 3).factorial)

        n.binom(3) * 6 * (n - 3).factorial = n * (n - 1) * (n - 2) * (n - 3).factorial
        (n - 3).factorial != 0
        6 * n.binom(3) = n * (n - 1) * (n - 2)
    }
}

/// When k > n, the binomial coefficient is zero.
theorem choose_out_of_bounds(n: Nat, k: Nat) {
    n < k implies n.binom(k) = 0
}

/// The binomial coefficient n choose k is positive whenever k is at most n.
theorem binom_pos(n: Nat, k: Nat) {
    k <= n implies 0 < n.binom(k)
} by {
    if k <= n {
        n.binom(k) * k.factorial * (n - k).factorial = n.factorial
        let p: Nat = k.factorial * (n - k).factorial
        n.binom(k) * p = n.factorial
        if n.binom(k) = 0 {
            n.binom(k) * p = 0
            false
        }
    }
}

/// Pascal's identity: each entry in Pascal's triangle is the sum of the two entries above it.
theorem pascal(n: Nat, k: Nat) {
    0 < k and k <= n implies n.suc.binom(k) = n.binom(k - 1) + n.binom(k)
} by {
    // Establish key facts about k
    let (k_pred: Nat) satisfy { k_pred.suc = k }

    // Establish bounds

    k.factorial * (n.suc - k).factorial != 0

    // LHS
    n.suc.binom(k) * (k.factorial * (n.suc - k).factorial) = n.suc.factorial

    // First term of RHS
    not n < k_pred
    n - k_pred = n.suc - k
    n.binom(k_pred) * k_pred.factorial * (n.suc - k).factorial = n.factorial
    k * n.binom(k_pred) = n.binom(k_pred) * k
    k * (n.binom(k_pred) * k_pred.factorial) = k * n.binom(k_pred) * k_pred.factorial
    k * (n.binom(k_pred) * k_pred.factorial * (n.suc - k).factorial) = k * (n.binom(k_pred) * k_pred.factorial) * (n.suc - k).factorial
    n.binom(k_pred) * k * k_pred.factorial * (n.suc - k).factorial = k * n.factorial

    // Second term of RHS
    n.binom(k) * k.factorial * (n - k).factorial = n.factorial
    n - k + 1 = (n - k).suc
    n - k + k = n
    (n - k).suc + k = (n - k + k).suc
    n.suc - k = (n - k).suc
    (n - k).suc * (n - k).factorial = (n - k).suc.factorial
    (n.suc - k).factorial = (n - k + 1) * (n - k).factorial
    n.binom(k) * k.factorial * (n - k + 1) = (n - k + 1) * (n.binom(k) * k.factorial)
    n.binom(k) * k.factorial * ((n - k + 1) * (n - k).factorial) = n.binom(k) * k.factorial * (n - k + 1) * (n - k).factorial
    n.binom(k) * k.factorial * (n.suc - k).factorial = (n - k + 1) * n.binom(k) * k.factorial * (n - k).factorial

    // Sum
    k * k_pred.factorial = k.factorial
    n.binom(k - 1) * (k.factorial * (n.suc - k).factorial) = n.binom(k - 1) * k.factorial * (n.suc - k).factorial
    n.binom(k) * (k.factorial * (n.suc - k).factorial) = n.binom(k) * k.factorial * (n.suc - k).factorial
    n.binom(k) * (k.factorial * (n - k).factorial) = n.binom(k) * k.factorial * (n - k).factorial
    n.binom(k_pred) * (k * k_pred.factorial) = n.binom(k_pred) * k * k_pred.factorial
    k * n.factorial = n.factorial * k
    n.binom(k) + n.binom(k - 1) = n.binom(k - 1) + n.binom(k)
    n.factorial * k + (n - k + 1) * n.factorial = (n - k + 1) * n.factorial + n.factorial * k
    (n.binom(k - 1) + n.binom(k)) * (k.factorial * (n.suc - k).factorial) = n.binom(k - 1) * (k.factorial * (n.suc - k).factorial) + n.binom(k) * (k.factorial * (n.suc - k).factorial)
    n.binom(k - 1) * (k.factorial * (n.suc - k).factorial) = k * n.factorial
    (n - k + 1) * (n.binom(k) * k.factorial * (n - k).factorial) = (n - k + 1) * n.factorial
    (n - k + 1) * (n.binom(k) * k.factorial * (n - k).factorial) = n.binom(k) * k.factorial * (n.suc - k).factorial
    n.binom(k) * (k.factorial * (n.suc - k).factorial) = (n - k + 1) * n.factorial
    n.binom(k - 1) * (k.factorial * (n.suc - k).factorial) + n.binom(k) * (k.factorial * (n.suc - k).factorial) = k * n.factorial + (n - k + 1) * n.factorial
    (n.binom(k - 1) + n.binom(k)) * (k.factorial * (n.suc - k).factorial) = k * n.factorial + (n - k + 1) * n.factorial
    (n.binom(k - 1) + n.binom(k)) * (k.factorial * (n.suc - k).factorial) = n.suc.factorial
    (n.binom(k - 1) + n.binom(k)) * (k.factorial * (n.suc - k).factorial) = n.suc.binom(k) * (k.factorial * (n.suc - k).factorial)
}

/// Pascal's identity in successor form, free of subtraction.
theorem pascal_suc(n: Nat, k: Nat) {
    k <= n implies n.suc.binom(k.suc) = n.binom(k) + n.binom(k.suc)
} by {
    if k <= n {
        if k = n {
            n.suc.binom(n.suc) = 1
            n.binom(n) = 1
            n.binom(n.suc) = 0
            n.suc.binom(k.suc) = n.binom(k) + n.binom(k.suc)
        } else {
            k < n
            k.suc <= n
            0 < k.suc
            pascal(n, k.suc)
            n.suc.binom(k.suc) = n.binom(k) + n.binom(k.suc)
        }
    }
}

// Helper function for the binomial theorem
define binomial_term(a: Nat, b: Nat, n: Nat, k: Nat) -> Nat {
    n.binom(k) * a.pow(k) * b.pow(n - k)
}

/// The boundary term at k=0 for the binomial expansion.
theorem binomial_term_zero(a: Nat, b: Nat, m: Nat) {
    binomial_term(a, b, m.suc, 0) = b * binomial_term(a, b, m, 0)
} by {

    binomial_term(a, b, m, 0) = b.pow(m)

}

/// The boundary term at k=m+1 for the binomial expansion.
theorem binomial_term_top(a: Nat, b: Nat, m: Nat) {
    binomial_term(a, b, m.suc, m.suc) = a * binomial_term(a, b, m, m)
} by {

    binomial_term(a, b, m, m) = a.pow(m)

}

/// Recursive relation for binomial terms: each term for n+1 equals a times the shifted
/// term for n plus b times the corresponding term for n.
theorem binomial_term_recurrence(a: Nat, b: Nat, m: Nat, k: Nat) {
    0 < k and k <= m implies
    binomial_term(a, b, m.suc, k) =
    a * binomial_term(a, b, m, k - 1) + b * binomial_term(a, b, m, k)
} by {
    // Establish k_pred for k - 1
    let (k_pred: Nat) satisfy { k_pred.suc = k }
    k_pred <= m

    // Expand LHS: binomial_term(a, b, m.suc, k)

    // Expand first term of RHS: a * binomial_term(a, b, m, k - 1)
    a.pow(k) = a * a.pow(k_pred)
    binomial_term(a, b, m, k_pred) = m.binom(k_pred) * a.pow(k_pred) * b.pow(m - k_pred)
    a.pow(k_pred) * m.binom(k_pred) = m.binom(k_pred) * a.pow(k_pred)
    a.pow(k) * m.binom(k_pred) = m.binom(k_pred) * a.pow(k)
    a * (m.binom(k_pred) * a.pow(k_pred) * b.pow(m - k_pred)) = a * (m.binom(k_pred) * a.pow(k_pred)) * b.pow(m - k_pred)
    m.binom(k_pred) * a.pow(k_pred) * b.pow(m - k_pred) * a =
        m.binom(k_pred) * a.pow(k_pred) * (b.pow(m - k_pred) * a)
    b.pow(m - k_pred) * a = a * b.pow(m - k_pred)
    m.binom(k_pred) * a.pow(k_pred) * (b.pow(m - k_pred) * a) =
        m.binom(k_pred) * a.pow(k_pred) * (a * b.pow(m - k_pred))
    m.binom(k_pred) * a.pow(k_pred) * (a * b.pow(m - k_pred)) =
        m.binom(k_pred) * (a.pow(k_pred) * a) * b.pow(m - k_pred)
    a.pow(k_pred) * a = a * a.pow(k_pred)
    m.binom(k_pred) * (a.pow(k_pred) * a) * b.pow(m - k_pred) =
        m.binom(k_pred) * (a * a.pow(k_pred)) * b.pow(m - k_pred)
    m.binom(k_pred) * (a * a.pow(k_pred)) * b.pow(m - k_pred) =
        m.binom(k_pred) * a.pow(k) * b.pow(m - k_pred)
    a * binomial_term(a, b, m, k_pred) =
        binomial_term(a, b, m, k_pred) * a
    a * binomial_term(a, b, m, k_pred) = m.binom(k_pred) * a.pow(k) * b.pow(m - k_pred)
    m - k_pred + k_pred = m
    (m - k_pred + k_pred).suc = m.suc
    m - k_pred + k_pred.suc = m.suc
    m - k_pred + k = m.suc
    m - k_pred = m.suc - k

    // Expand second term of RHS: b * binomial_term(a, b, m, k)
    binomial_term(a, b, m, k) = m.binom(k) * a.pow(k) * b.pow(m - k)
    b * binomial_term(a, b, m, k) =
        binomial_term(a, b, m, k) * b
    b * binomial_term(a, b, m, k) =
        m.binom(k) * a.pow(k) * b.pow(m - k) * b
    m.binom(k) * a.pow(k) * b.pow(m - k) * b =
        m.binom(k) * a.pow(k) * (b.pow(m - k) * b)
    b.pow(m - k) * b = b * b.pow(m - k)
    b * binomial_term(a, b, m, k) = m.binom(k) * a.pow(k) * (b * b.pow(m - k))
    b.pow(m.suc - k) = b.pow(m - k + 1)

    // Combine the two terms using the distributive property
    let rhs_sum = a * binomial_term(a, b, m, k - 1) + b * binomial_term(a, b, m, k)
    rhs_sum = (m.binom(k - 1) + m.binom(k)) * a.pow(k) * b.pow(m.suc - k)

    // Apply Pascal's identity
    m.binom(k - 1) + m.binom(k) = m.suc.binom(k)

    // Connect LHS and RHS
}

/// Alternative form of the binomial term recurrence, avoiding subtraction by using k.suc.
theorem alt_binomial_term_recurrence(a: Nat, b: Nat, m: Nat, k: Nat) {
    k < m implies
    binomial_term(a, b, m.suc, k.suc) =
    a * binomial_term(a, b, m, k) + b * binomial_term(a, b, m, k.suc)
} by {
    // We have k < m, which gives us k.suc <= m and 0 < k.suc
    binomial_term(a, b, m.suc, k.suc) = a * binomial_term(a, b, m, k.suc - 1) + b * binomial_term(a, b, m, k.suc)
}

/// Helper: the middle partial sum splits into two parts using the recurrence.
theorem binomial_middle_sum(a: Nat, b: Nat, m: Nat) {
    partial(compose(binomial_term(a, b, m.suc), Nat.suc), m) =
        a * partial(binomial_term(a, b, m), m) + b * partial(compose(binomial_term(a, b, m), Nat.suc), m)
} by {
    // Define helper functions for clarity
    define f(k: Nat) -> Nat { binomial_term(a, b, m, k) }
    define g(k: Nat) -> Nat { compose(binomial_term(a, b, m), Nat.suc)(k) }
    define h(k: Nat) -> Nat { compose(binomial_term(a, b, m.suc), Nat.suc)(k) }

    // Show pointwise equality: for each k < m, h(k) = a * f(k) + b * g(k)
    forall(k: Nat) {
        if k < m {
            // h(k) = binomial_term(a, b, m.suc, k.suc)
            h(k) = binomial_term(a, b, m.suc, k.suc)

            // Use alt_binomial_term_recurrence
            binomial_term(a, b, m.suc, k.suc) = a * binomial_term(a, b, m, k) + b * binomial_term(a, b, m, k.suc)
            h(k) = a * binomial_term(a, b, m, k) + b * binomial_term(a, b, m, k.suc)

            // Simplify RHS
            h(k) = a * f(k) + b * g(k)
            h(k) = mul_fn(a, f)(k) + mul_fn(b, g)(k)
            h(k) = add_fn(mul_fn(a, f), mul_fn(b, g), k)
        }
    }

    // Use partial_pointwise_eq
    partial(h, m) = partial(add_fn(mul_fn(a, f), mul_fn(b, g)), m)

    // Use partial_add to split the sum
    partial(h, m) = partial(mul_fn(a, f), m) + partial(mul_fn(b, g), m)

    // Use partial_scalar_mul to factor out the scalars
    partial(h, m) = a * partial(f, m) + b * partial(g, m)
}

/// Distributing (a + b) across a partial sum of binomial terms gives the next row.
theorem binomial_distribution(a: Nat, b: Nat, m: Nat) {
    (a + b) * partial(binomial_term(a, b, m), m.suc) = partial(binomial_term(a, b, m.suc), m.suc.suc)
} by {
    // LHS: Expand using distributivity

    // RHS: Peel off the last term using partial_split_last

    // Simplify the last term

    // Now peel off the first term using partial_shift_suc

    // Simplify the first term

    // Use binomial_middle_sum to split the middle partial sum

    // Build up the RHS step by step
    // Start with: RHS = first + middle + last
    let rhs_part1 = b * binomial_term(a, b, m, 0) + partial(compose(binomial_term(a, b, m.suc), Nat.suc), m)

    // Expand the middle part using binomial_middle_sum

    // Combine the 'b' terms using partial_shift_suc
    binomial_term(a, b, m, 0) + partial(compose(binomial_term(a, b, m), Nat.suc), m) = partial(binomial_term(a, b, m), m.suc)
    b * binomial_term(a, b, m, 0) + (b * partial(compose(binomial_term(a, b, m), Nat.suc), m) + a * partial(binomial_term(a, b, m), m)) = b * binomial_term(a, b, m, 0) + b * partial(compose(binomial_term(a, b, m), Nat.suc), m) + a * partial(binomial_term(a, b, m), m)

    // So rhs_part1 = b * partial(...) + a * partial(..., m)

    // Therefore the full RHS is

    // Combine the 'a' terms using partial_split_last
    a * partial(binomial_term(a, b, m), m) + a * binomial_term(a, b, m, m) = a * partial(binomial_term(a, b, m), m.suc)
    binomial_term(a, b, m.suc, 0) + partial(compose(binomial_term(a, b, m.suc), Nat.suc), m) = partial(binomial_term(a, b, m.suc), m.suc)
    b * partial(binomial_term(a, b, m), m.suc) + (a * partial(binomial_term(a, b, m), m) + a * binomial_term(a, b, m, m)) = b * partial(binomial_term(a, b, m), m.suc) + a * partial(binomial_term(a, b, m), m) + a * binomial_term(a, b, m, m)

    // Final result
}

/// The binomial theorem: (a + b)^n equals the sum of binomial terms.
theorem binomial(a: Nat, b: Nat, n: Nat) {
    (a + b).pow(n) = partial(binomial_term(a, b, n), n.suc)
} by {
    define f(x: Nat) -> Bool {
        (a + b).pow(x) = partial(binomial_term(a, b, x), x.suc)
    }

    // Base case: n = 0
    partial(binomial_term(a, b, 0), 1) = binomial_term(a, b, 0, 0)
    binomial_term(a, b, 0, 0) = Nat.0.binom(0) * a.pow(0) * b.pow(0 - 0)
    Nat.0.binom(0) = 1
    a.pow(0) = 1
    b.pow(0) = 1
    Nat.0 - Nat.0 = Nat.0
    (a + b).pow(0) = 1
    (a + b).pow(0) = partial(binomial_term(a, b, 0), 0.suc)

    // Inductive step
    forall(m: Nat) {
        if f(m) {
            // Induction hypothesis
            partial(binomial_term(a, b, m), m.suc) = (a + b).pow(m)

            // LHS: (a + b)^(m+1) = (a + b) * (a + b)^m

            // Apply binomial_distribution to complete the inductive step
            (a + b) * partial(binomial_term(a, b, m), m.suc) = partial(binomial_term(a, b, m.suc), m.suc.suc)

            f(m.suc)
        }
    }

    f(n)
}

/// Absorption identity: k * (n choose k) = n * (n - 1 choose k - 1) when 0 < k <= n.
theorem binom_absorption(n: Nat, k: Nat) {
    0 < k and k <= n implies k * n.binom(k) = n * (n - 1).binom(k - 1)
} by {
    if 0 < k and k <= n {
        let (k_pred: Nat) satisfy { k_pred.suc = k }
        let (n_pred: Nat) satisfy { n_pred.suc = n }
        k_pred <= n_pred
        not n_pred < k_pred

        // Expand n.binom(k) equation
        n.binom(k) * k.factorial * (n - k).factorial = n.factorial
        k.factorial = k * k_pred.factorial
        n.factorial = n * n_pred.factorial
        n.binom(k) * (k * k_pred.factorial) * (n - k).factorial = n * n_pred.factorial
        k * n.binom(k) * (k_pred.factorial * (n - k).factorial) = n * n_pred.factorial

        // Expand (n-1).binom(k-1) equation
        n_pred - k_pred + k_pred = n_pred
        n_pred - k_pred + k_pred.suc = n_pred.suc
        n_pred - k_pred + k = n
        n_pred - k_pred = n - k
        n_pred.binom(k_pred) * k_pred.factorial * (n - k).factorial = n_pred.factorial

        // Cancel
        let p: Nat = k_pred.factorial * (n - k).factorial
        p != 0
        k * n.binom(k) * p = n * n_pred.binom(k_pred) * p
        n_pred.binom(k_pred) = (n - 1).binom(k - 1)
        k * n.binom(k) = n * (n - 1).binom(k - 1)
    }
}

/// Absorption identity in successor form, free of subtraction.
theorem binom_absorption_suc(n: Nat, k: Nat) {
    k <= n implies k.suc * n.suc.binom(k.suc) = n.suc * n.binom(k)
} by {
    if k <= n {
        k.suc <= n.suc
        binom_absorption(n.suc, k.suc)
        k.suc * n.suc.binom(k.suc) = n.suc * (n.suc - 1).binom(k.suc - 1)
        k.suc * n.suc.binom(k.suc) = n.suc * n.binom(k)
    }
}

/// Complementary absorption identity: (n - k) * binom(n, k) = (k + 1) * binom(n, k + 1) when k < n.
theorem binom_complement_absorption(n: Nat, k: Nat) {
    k < n implies (n - k) * n.binom(k) = k.suc * n.binom(k.suc)
} by {
    if k < n {
        k <= n
        not n < k
        k.suc <= n
        not n < k.suc
        0 < n - k
        let m: Nat satisfy { m.suc = n - k }
        m = n - k - 1
        m.suc * m.factorial = m.suc.factorial
        (n - k) * m.factorial = (n - k).factorial

        // n - k.suc = m
        n - k.suc + k.suc = n
        m + k.suc = n - k + k
        m = n - k.suc

        n.binom(k) * k.factorial * (n - k).factorial = n.factorial
        n.binom(k.suc) * k.suc.factorial * (n - k.suc).factorial = n.factorial
        n.binom(k.suc) * k.suc.factorial * m.factorial = n.factorial

        k.suc.factorial = k.suc * k.factorial
        n.binom(k.suc) * (k.suc * k.factorial) * m.factorial = n.factorial
        k.suc * n.binom(k.suc) * k.factorial * m.factorial = n.factorial

        n.binom(k) * k.factorial * ((n - k) * m.factorial) = n.factorial
        let q1: Nat = n.binom(k) * k.factorial
        q1 * ((n - k) * m.factorial) = n.factorial
        q1 * ((n - k) * m.factorial) = q1 * (n - k) * m.factorial
        q1 * (n - k) = (n - k) * q1
        q1 * (n - k) * m.factorial = (n - k) * q1 * m.factorial
        (n - k) * q1 = (n - k) * (n.binom(k) * k.factorial)
        (n - k) * (n.binom(k) * k.factorial) = (n - k) * n.binom(k) * k.factorial
        (n - k) * q1 * m.factorial = (n - k) * n.binom(k) * k.factorial * m.factorial
        (n - k) * n.binom(k) * k.factorial * m.factorial = n.factorial

        let p: Nat = k.factorial * m.factorial
        k.factorial != 0
        m.factorial != 0
        p != 0
        (n - k) * n.binom(k) * p = (n - k) * n.binom(k) * (k.factorial * m.factorial)
        (n - k) * n.binom(k) * (k.factorial * m.factorial) = (n - k) * n.binom(k) * k.factorial * m.factorial
        (n - k) * n.binom(k) * p = n.factorial
        k.suc * n.binom(k.suc) * p = k.suc * n.binom(k.suc) * (k.factorial * m.factorial)
        k.suc * n.binom(k.suc) * (k.factorial * m.factorial) = k.suc * n.binom(k.suc) * k.factorial * m.factorial
        k.suc * n.binom(k.suc) * p = n.factorial
        (n - k) * n.binom(k) * p = k.suc * n.binom(k.suc) * p
        (n - k) * n.binom(k) = k.suc * n.binom(k.suc)
    }
}

/// Binomial coefficients are weakly increasing in the left half: when 2 * (k + 1) is
/// at most n, n choose k is at most n choose k + 1.
theorem binom_le_binom_suc(n: Nat, k: Nat) {
    2 * k.suc <= n implies n.binom(k) <= n.binom(k.suc)
} by {
    if 2 * k.suc <= n {
        2 * k.suc = k.suc + k.suc
        k.suc + k.suc <= n
        k.suc <= n
        not n < k.suc
        k < n
        binom_complement_absorption(n, k)
        // Establish k.suc <= n - k.
        let m: Nat satisfy { k.suc + k.suc + m = n }
        k + 1 = k.suc
        k + (k.suc + m + 1) = k + k.suc + m + 1
        k + k.suc + m + 1 = k.suc + k + m + 1
        k.suc + k + m + 1 = k.suc + (k + 1) + m
        k.suc + (k + 1) + m = k.suc + k.suc + m
        k + (k.suc + m + 1) = n
        k.suc + m + 1 = n - k
        k.suc <= k.suc + m + 1
        k.suc <= n - k
        // Multiply binom(n, k) by both sides.
        let b: Nat = n.binom(k)
        let b1: Nat = n.binom(k.suc)
        (n - k) * b = k.suc * b1
        // Multiplying k.suc <= n - k by b on the right.
        let d: Nat satisfy { k.suc + d = n - k }
        k.suc * b + d * b = (k.suc + d) * b
        (k.suc + d) * b = (n - k) * b
        k.suc * b + d * b = (n - k) * b
        k.suc * b <= (n - k) * b
        k.suc * b <= k.suc * b1
        // Cancel k.suc on the left using a contradiction argument.
        if b1 < b {
            b1.suc <= b
            let e: Nat satisfy { b1.suc + e = b }
            k.suc * b1.suc + k.suc * e = k.suc * (b1.suc + e)
            k.suc * (b1.suc + e) = k.suc * b
            k.suc * b1.suc = k.suc * b1 + k.suc
            k.suc * b1 + k.suc + k.suc * e = k.suc * b
            k.suc * b <= k.suc * b1
            k.suc * b1 + (k.suc + k.suc * e) = k.suc * b1 + k.suc + k.suc * e
            k.suc * b1 + (k.suc + k.suc * e) <= k.suc * b1
            k.suc + k.suc * e <= 0
            k.suc <= 0
            false
        }
        b <= b1
        n.binom(k) <= n.binom(k.suc)
    }
}

/// Binomial coefficients are weakly decreasing in the right half: when n is at most 2 * k,
/// n choose k + 1 is at most n choose k.
theorem binom_suc_le_binom(n: Nat, k: Nat) {
    n <= 2 * k implies n.binom(k.suc) <= n.binom(k)
} by {
    if n <= 2 * k {
        if n < k.suc {
            n.binom(k.suc) = 0
            n.binom(k.suc) <= n.binom(k)
        } else {
            k.suc <= n
            k <= n
            not n < k
            not n < k.suc
            // Use symmetry: binom(n, k.suc) = binom(n, n - k.suc), binom(n, k) = binom(n, n - k).
            choose_symm_sub_form(n, k.suc)
            choose_symm_sub_form(n, k)
            n.binom(k.suc) = n.binom(n - k.suc)
            n.binom(k) = n.binom(n - k)
            // n - k.suc + 1 = n - k.
            n - k.suc + k.suc = n
            n - k.suc + k + 1 = n
            n - k.suc + 1 + k = n
            n - k.suc + 1 = n - k
            (n - k.suc).suc = n - k
            // Apply binom_le_binom_suc with j = n - k.suc.
            // Need 2 * (n - k.suc).suc <= n, i.e., 2 * (n - k) <= n.
            2 * k = k + k
            n <= k + k
            let r: Nat satisfy { n + r = k + k }
            n - k + n - k = n - k + n - k
            (n - k) + k = n
            // (n - k) + (n - k) + r = (n - k + k) - k + n = ? simpler approach
            (n - k) + (n - k) + r = (n - k) + ((n - k) + r)
            (n - k) + r + k = (n - k) + k + r
            (n - k) + k + r = n + r
            (n - k) + r + k = k + k
            (n - k) + ((n - k) + r) = (n - k) + k
            2 * (n - k) + r = n
            binom_le_binom_suc(n, n - k.suc)
            n.binom(n - k.suc) <= n.binom((n - k.suc).suc)
            n.binom(k.suc) <= n.binom(k)
        }
    }
}

/// Subset selection identity: choosing a k-subset and then a j-subset of it is the same
/// as choosing the j-subset first and then completing it to a k-subset.
theorem binom_subset_product(n: Nat, k: Nat, j: Nat) {
    j <= k and k <= n implies n.binom(k) * k.binom(j) = n.binom(j) * (n - j).binom(k - j)
} by {
    if j <= k and k <= n {
        j <= n
        not n < k
        not k < j
        not n < j
        let a: Nat satisfy { j + a = k }
        let b: Nat satisfy { k + b = n }
        a = k - j
        b = n - k
        j + a + b = n
        j + (a + b) = n
        a + b = n - j
        k - j + (n - k) = n - j
        k - j <= n - j
        not (n - j) < (k - j)
        (n - j) - (k - j) + (k - j) = n - j
        (n - j) - (k - j) + (k - j) = (n - k) + (k - j)
        (n - j) - (k - j) = n - k

        // Factorial relations
        n.binom(k) * k.factorial * (n - k).factorial = n.factorial
        k.binom(j) * j.factorial * (k - j).factorial = k.factorial
        n.binom(j) * j.factorial * (n - j).factorial = n.factorial
        (n - j).binom(k - j) * (k - j).factorial * (n - k).factorial = (n - j).factorial

        let p: Nat = j.factorial * (k - j).factorial * (n - k).factorial
        j.factorial != 0
        (k - j).factorial != 0
        (n - k).factorial != 0
        p != 0

        // LHS * p
        let bk: Nat = n.binom(k)
        let bj: Nat = k.binom(j)
        let jf: Nat = j.factorial
        let kjf: Nat = (k - j).factorial
        let nkf: Nat = (n - k).factorial
        p = jf * kjf * nkf
        bj * jf * kjf = k.factorial
        bk * k.factorial * nkf = n.factorial
        bk * (bj * jf * kjf) * nkf = bk * k.factorial * nkf
        bk * (bj * jf * kjf) * nkf = n.factorial
        bk * bj * jf * kjf * nkf = bk * (bj * jf * kjf) * nkf
        bk * bj * jf * kjf * nkf = n.factorial
        bk * bj * p = bk * bj * (jf * kjf * nkf)
        bk * bj * (jf * kjf * nkf) = bk * bj * jf * kjf * nkf
        bk * bj * p = n.factorial
        n.binom(k) * k.binom(j) * p = n.factorial

        // RHS * p
        let cj: Nat = n.binom(j)
        let cnjkj: Nat = (n - j).binom(k - j)
        let njf: Nat = (n - j).factorial
        cj * jf * njf = n.factorial
        cnjkj * kjf * nkf = njf
        cj * jf * (cnjkj * kjf * nkf) = cj * jf * njf
        cj * jf * (cnjkj * kjf * nkf) = n.factorial
        cnjkj * jf = jf * cnjkj
        cj * (cnjkj * jf) = cj * (jf * cnjkj)
        cj * (cnjkj * jf) = cj * cnjkj * jf
        cj * (jf * cnjkj) = cj * jf * cnjkj
        cj * cnjkj * jf = cj * jf * cnjkj
        cj * cnjkj * jf * kjf = cj * jf * cnjkj * kjf
        cj * cnjkj * jf * kjf * nkf = cj * jf * cnjkj * kjf * nkf
        cj * jf * cnjkj * kjf = cj * jf * (cnjkj * kjf)
        cj * jf * (cnjkj * kjf) * nkf = cj * jf * (cnjkj * kjf) * nkf
        cj * jf * (cnjkj * kjf) * nkf = cj * jf * ((cnjkj * kjf) * nkf)
        (cnjkj * kjf) * nkf = cnjkj * kjf * nkf
        cj * jf * cnjkj * kjf * nkf = cj * jf * (cnjkj * kjf * nkf)
        cj * cnjkj * jf * kjf * nkf = cj * jf * (cnjkj * kjf * nkf)
        cj * cnjkj * jf * kjf * nkf = n.factorial
        cj * cnjkj * p = cj * cnjkj * (jf * kjf * nkf)
        cj * cnjkj * (jf * kjf * nkf) = cj * cnjkj * jf * kjf * nkf
        cj * cnjkj * p = n.factorial
        n.binom(j) * (n - j).binom(k - j) * p = n.factorial

        n.binom(k) * k.binom(j) * p = n.binom(j) * (n - j).binom(k - j) * p
        n.binom(k) * k.binom(j) = n.binom(j) * (n - j).binom(k - j)
    }
}

/// Pascal's identity in successor form, valid for all k and n without any bound.
theorem pascal_suc_unbounded(n: Nat, k: Nat) {
    n.suc.binom(k.suc) = n.binom(k) + n.binom(k.suc)
} by {
    if k <= n {
        pascal_suc(n, k)
        n.suc.binom(k.suc) = n.binom(k) + n.binom(k.suc)
    } else {
        n < k
        choose_out_of_bounds(n, k)
        choose_out_of_bounds(n, k.suc)
        n.binom(k.suc) = Nat.0
        n.suc < k.suc
        choose_out_of_bounds(n.suc, k.suc)
        n.suc.binom(k.suc) = n.binom(k) + n.binom(k.suc)
    }
}

/// The j-th term in Vandermonde's convolution for choosing k items from m + n.
define vandermonde_term(m: Nat, n: Nat, k: Nat, j: Nat) -> Nat {
    m.binom(j) * n.binom(k - j)
}

/// Auxiliary lemma: the partial sum of vandermonde_term at m = 0 equals n.binom(k)
/// for any positive upper bound, since only the j = 0 term is nonzero.
theorem vandermonde_zero_aux(n: Nat, k: Nat, m: Nat) {
    partial(vandermonde_term(Nat.0, n, k), m.suc) = n.binom(k)
} by {
    define p(x: Nat) -> Bool {
        partial(vandermonde_term(Nat.0, n, k), x.suc) = n.binom(k)
    }
    // Base: x = 0
    partial_one[Nat](vandermonde_term(Nat.0, n, k))
    partial(vandermonde_term(Nat.0, n, k), 1) = vandermonde_term(Nat.0, n, k, Nat.0)
    vandermonde_term(Nat.0, n, k, Nat.0) = Nat.0.binom(Nat.0) * n.binom(k - Nat.0)
    Nat.0.binom(Nat.0) = 1
    k - Nat.0 = k
    vandermonde_term(Nat.0, n, k, Nat.0) = n.binom(k)
    p(Nat.0)
    forall(x: Nat) {
        if p(x) {
            partial_split_last[Nat](vandermonde_term(Nat.0, n, k), x.suc)
            partial(vandermonde_term(Nat.0, n, k), x.suc.suc) =
                partial(vandermonde_term(Nat.0, n, k), x.suc) + vandermonde_term(Nat.0, n, k, x.suc)
            Nat.0 < x.suc
            vandermonde_term(Nat.0, n, k, x.suc) = Nat.0.binom(x.suc) * n.binom(k - x.suc)
            choose_out_of_bounds(Nat.0, x.suc)
            Nat.0.binom(x.suc) = Nat.0
            vandermonde_term(Nat.0, n, k, x.suc) = Nat.0
            partial(vandermonde_term(Nat.0, n, k), x.suc) = n.binom(k)
            partial(vandermonde_term(Nat.0, n, k), x.suc.suc) = n.binom(k)
            p(x.suc)
        }
    }
    alt_induction(p)
    forall(x: Nat) { p(x) }
}

/// Vandermonde's identity, inductive step: assuming Vandermonde holds for big_m at every k,
/// it holds for big_m.suc at every k1.
theorem vandermonde_step(big_m: Nat, n: Nat, k1: Nat) {
    (forall(kk: Nat) {
        partial(vandermonde_term(big_m, n, kk), kk.suc) = (big_m + n).binom(kk)
    }) implies partial(vandermonde_term(big_m.suc, n, k1), k1.suc) = (big_m.suc + n).binom(k1)
} by {
    if forall(kk: Nat) {
        partial(vandermonde_term(big_m, n, kk), kk.suc) = (big_m + n).binom(kk)
    } {
        if k1 = Nat.0 {
                    partial_one[Nat](vandermonde_term(big_m.suc, n, Nat.0))
                    big_m.suc.binom(Nat.0) = 1
                    n.binom(Nat.0) = 1
                    (big_m.suc + n).binom(Nat.0) = 1
                    partial(vandermonde_term(big_m.suc, n, k1), k1.suc) = (big_m.suc + n).binom(k1)
                } else {
                    let big_k: Nat satisfy { big_k.suc = k1 }

                    // Use partial_drop_first to peel off the j = 0 term
                    Nat.0 < k1.suc
                    partial_drop_first[Nat](vandermonde_term(big_m.suc, n, k1), k1.suc)
                    k1.suc - 1 = k1
                    partial(vandermonde_term(big_m.suc, n, k1), k1.suc) =
                        vandermonde_term(big_m.suc, n, k1, Nat.0) + partial(compose(vandermonde_term(big_m.suc, n, k1), Nat.suc), k1)
                    vandermonde_term(big_m.suc, n, k1, Nat.0) = big_m.suc.binom(Nat.0) * n.binom(k1 - Nat.0)
                    big_m.suc.binom(Nat.0) = 1
                    k1 - Nat.0 = k1
                    vandermonde_term(big_m.suc, n, k1, Nat.0) = n.binom(k1)

                    // Define helper functions for the two halves of pascal expansion.
                    define g1(i: Nat) -> Nat { vandermonde_term(big_m, n, big_k, i) }
                    define g2(i: Nat) -> Nat { compose(vandermonde_term(big_m, n, k1), Nat.suc, i) }

                    // Show pointwise: compose(vandermonde_term(big_m.suc, n, k1), Nat.suc)(i) = add_fn(g1, g2, i) for i < k1
                    forall(i: Nat) {
                        if i < k1 {
                            i <= big_k
                            big_k - i + i = big_k
                            big_k - i + i.suc = big_k.suc
                            pascal_suc_unbounded(big_m, i)
                            big_m.suc.binom(i.suc) * n.binom(k1 - i.suc) =
                                big_m.binom(i) * n.binom(k1 - i.suc) + big_m.binom(i.suc) * n.binom(k1 - i.suc)
                            big_m.binom(i) * n.binom(k1 - i.suc) = big_m.binom(i) * n.binom(big_k - i)
                            g2(i) = compose(vandermonde_term(big_m, n, k1), Nat.suc, i)
                            add_fn(g1, g2, i) = g1(i) + g2(i)
                            compose(vandermonde_term(big_m.suc, n, k1), Nat.suc, i) = add_fn(g1, g2, i)
                        }
                    }

                    // Sum equality via pointwise_eq
                    partial_pointwise_eq[Nat](compose(vandermonde_term(big_m.suc, n, k1), Nat.suc), add_fn(g1, g2), k1)
                    partial(compose(vandermonde_term(big_m.suc, n, k1), Nat.suc), k1) = partial(add_fn(g1, g2), k1)

                    // Split sum via partial_add
                    partial_add[Nat](g1, g2, k1)

                    // Apply IH to g1: partial(g1, big_k.suc) = (big_m + n).binom(big_k)
                    forall(i: Nat) {
                        if i < k1 {
                            g1(i) = vandermonde_term(big_m, n, big_k, i)
                        }
                    }
                    partial_pointwise_eq[Nat](g1, vandermonde_term(big_m, n, big_k), k1)

                    // Apply IH to g2 via partial_drop_first inversion.
                    partial_drop_first[Nat](vandermonde_term(big_m, n, k1), k1.suc)
                    partial(vandermonde_term(big_m, n, k1), k1.suc) =
                        vandermonde_term(big_m, n, k1, Nat.0) + partial(compose(vandermonde_term(big_m, n, k1), Nat.suc), k1)
                    big_m.binom(Nat.0) = 1

                    // Combine
                    let aa: Nat = n.binom(k1)
                    let bb: Nat = (big_m + n).binom(big_k)
                    let cc: Nat = partial(g2, k1)
                    (big_m + n).binom(big_k) + (n.binom(k1) + partial(g2, k1)) =
                        (big_m + n).binom(big_k) + (big_m + n).binom(k1)

                    // Apply pascal_suc_unbounded to (big_m + n) at big_k
                    pascal_suc_unbounded(big_m + n, big_k)
            partial(vandermonde_term(big_m.suc, n, k1), k1.suc) = (big_m.suc + n).binom(k1)
        }
    }
}

/// Induction predicate for Vandermonde's identity at a fixed pair `(m, n)`.
define vandermonde_pred(m: Nat, n: Nat) -> Bool {
    forall(k: Nat) {
        partial(vandermonde_term(m, n, k), k.suc) = (m + n).binom(k)
    }
}

lemma vandermonde_pred_zero(n: Nat) {
    vandermonde_pred(Nat.0, n)
} by {
    forall(k: Nat) {
        vandermonde_zero_aux(n, k, k)
        partial(vandermonde_term(Nat.0, n, k), k.suc) = (Nat.0 + n).binom(k)
    }
}

lemma vandermonde_pred_suc(m: Nat, n: Nat) {
    vandermonde_pred(m, n) implies vandermonde_pred(m.suc, n)
} by {
    if vandermonde_pred(m, n) {
        if forall(kk: Nat) {
            partial(vandermonde_term(m, n, kk), kk.suc) = (m + n).binom(kk)
        } {
            forall(k1: Nat) {
                vandermonde_step(m, n, k1)
                partial(vandermonde_term(m.suc, n, k1), k1.suc) = (m.suc + n).binom(k1)
            }
        }
        if not forall(kk: Nat) {
            partial(vandermonde_term(m, n, kk), kk.suc) = (m + n).binom(kk)
        } {
            false
        }
        vandermonde_pred(m.suc, n)
    }
}

lemma vandermonde_pred_all(n: Nat) {
    forall(m: Nat) {
        vandermonde_pred(m, n)
    }
} by {
    define p(m: Nat) -> Bool {
        vandermonde_pred(m, n)
    }
    vandermonde_pred_zero(n)
    p(Nat.0)
    forall(m: Nat) {
        if p(m) {
            vandermonde_pred_suc(m, n)
            p(m.suc)
        }
    }
    alt_induction(p)
    forall(m: Nat) { p(m) }
    forall(m: Nat) {
        vandermonde_pred(m, n)
    }
}

/// Vandermonde's identity for binomial coefficients.
theorem vandermonde(m: Nat, n: Nat, k: Nat) {
    partial(vandermonde_term(m, n, k), k.suc) = (m + n).binom(k)
} by {
    vandermonde_pred_all(n)
    if forall(kk: Nat) {
        partial(vandermonde_term(m, n, kk), kk.suc) = (m + n).binom(kk)
    } {
        partial(vandermonde_term(m, n, k), k.suc) = (m + n).binom(k)
    }
    if not forall(kk: Nat) {
        partial(vandermonde_term(m, n, kk), kk.suc) = (m + n).binom(kk)
    } {
        false
    }
}

/// The square of the `j`th binomial coefficient in row `n`.
define binom_square_term(n: Nat, j: Nat) -> Nat {
    n.binom(j) * n.binom(j)
}

/// The sum of the squares of the binomial coefficients in row `n` is the
/// central binomial coefficient in row `2*n`.
theorem binom_square_sum(n: Nat) {
    partial(binom_square_term(n), n.suc) = (n + n).binom(n)
} by {
    vandermonde(n, n, n)
    partial(vandermonde_term(n, n, n), n.suc) = (n + n).binom(n)
    forall(j: Nat) {
        if j < n.suc {
            j <= n
            choose_symm_sub_form(n, j)
            n.binom(j) = n.binom(n - j)
            vandermonde_term(n, n, n, j) =
                n.binom(j) * n.binom(n - j)
            binom_square_term(n, j) = n.binom(j) * n.binom(j)
            vandermonde_term(n, n, n, j) = binom_square_term(n, j)
        }
    }
    partial_pointwise_eq[Nat](
        vandermonde_term(n, n, n), binom_square_term(n), n.suc)
    partial(vandermonde_term(n, n, n), n.suc) =
        partial(binom_square_term(n), n.suc)
    partial(binom_square_term(n), n.suc) = (n + n).binom(n)
}

/// The `j`th term in the hockey-stick sum with fixed lower index `r`.
define hockey_stick_term(r: Nat, j: Nat) -> Nat {
    (r + j).binom(r)
}

/// The hockey-stick identity for binomial coefficients.
theorem hockey_stick(r: Nat, n: Nat) {
    partial(hockey_stick_term(r), n.suc) = (r + n.suc).binom(r.suc)
} by {
    define p(k: Nat) -> Bool {
        partial(hockey_stick_term(r), k.suc) = (r + k.suc).binom(r.suc)
    }
    partial_one[Nat](hockey_stick_term(r))
    partial(hockey_stick_term(r), Nat.1) = hockey_stick_term(r, Nat.0)
    hockey_stick_term(r, Nat.0) = (r + Nat.0).binom(r)
    r + Nat.0 = r
    choose_n(r)
    r.binom(r) = Nat.1
    r + Nat.0.suc = r.suc
    choose_n(r.suc)
    r.suc.binom(r.suc) = Nat.1
    p(Nat.0)
    forall(k: Nat) {
        if p(k) {
            partial_split_last[Nat](hockey_stick_term(r), k.suc)
            partial(hockey_stick_term(r), k.suc.suc) =
                partial(hockey_stick_term(r), k.suc) +
                    hockey_stick_term(r, k.suc)
            hockey_stick_term(r, k.suc) = (r + k.suc).binom(r)
            partial(hockey_stick_term(r), k.suc) =
                (r + k.suc).binom(r.suc)
            pascal_suc_unbounded(r + k.suc, r)
            (r + k.suc).suc.binom(r.suc) =
                (r + k.suc).binom(r) + (r + k.suc).binom(r.suc)
            (r + k.suc).binom(r.suc) + (r + k.suc).binom(r) =
                (r + k.suc).binom(r) + (r + k.suc).binom(r.suc)
            r + k.suc.suc = (r + k.suc).suc
            partial(hockey_stick_term(r), k.suc.suc) =
                (r + k.suc.suc).binom(r.suc)
            p(k.suc)
        }
    }
    p(Nat.0) and forall(k: Nat) { p(k) implies p(k.suc) }
    alt_induction(p)
    forall(k: Nat) { p(k) }
    p(n)
}

/// The sum of a row of binomial coefficients equals 2 to the n.
theorem binom_row_sum(n: Nat) {
    partial(n.binom, n.suc) = 2.pow(n)
} by {
    binomial(1, 1, n)
    forall(k: Nat) {
        binomial_term(1, 1, n, k) = n.binom(k)
    }
}

/// The `k`th coefficient in the first moment of binomial row `n`.
define binom_weighted_term(n: Nat, k: Nat) -> Nat {
    k * n.binom(k)
}

/// The first moment of binomial row `n` is `n * 2^(n - 1)`.
theorem binom_weighted_row_sum(n: Nat) {
    partial(binom_weighted_term(n), n.suc) = n * 2.pow(n - 1)
} by {
    if n = Nat.0 {
        partial_one[Nat](binom_weighted_term(Nat.0))
        binom_weighted_term(Nat.0, Nat.0) = Nat.0
        partial(binom_weighted_term(Nat.0), Nat.1) = Nat.0
        Nat.0 * 2.pow(Nat.0 - Nat.1) = Nat.0
        partial(binom_weighted_term(n), n.suc) = n * 2.pow(n - Nat.1)
    } else {
        let m: Nat satisfy {
            m.suc = n
        }
        m = n - Nat.1
        partial_drop_first[Nat](binom_weighted_term(n), n.suc)
        binom_weighted_term(n, Nat.0) = Nat.0
        partial(binom_weighted_term(n), n.suc) =
            partial(compose(binom_weighted_term(n), Nat.suc), n)

        forall(k: Nat) {
            if k < n {
                k <= m
                binom_absorption_suc(m, k)
                k.suc * m.suc.binom(k.suc) = m.suc * m.binom(k)
                binom_weighted_term(n, k.suc) = n * m.binom(k)
                compose(binom_weighted_term(n), Nat.suc, k) = mul_fn(n, m.binom, k)
            }
        }
        partial_pointwise_eq[Nat](
            compose(binom_weighted_term(n), Nat.suc), mul_fn(n, m.binom), n)
        partial(compose(binom_weighted_term(n), Nat.suc), n) =
            partial(mul_fn(n, m.binom), n)
        partial_scalar_mul[Nat](n, m.binom, n)
        n * partial(m.binom, n) = partial(mul_fn(n, m.binom), n)
        binom_row_sum(m)
        partial(m.binom, m.suc) = 2.pow(m)
        partial(m.binom, n) = 2.pow(n - Nat.1)
        partial(binom_weighted_term(n), n.suc) = n * 2.pow(n - Nat.1)
    }
    partial(binom_weighted_term(n), n.suc) = n * 2.pow(n - Nat.1)
}

/// The `k`th coefficient in the second falling-factorial moment of binomial row `n`.
define binom_second_weighted_term(n: Nat, k: Nat) -> Nat {
    k * (k - Nat.1) * n.binom(k)
}

/// The second falling-factorial moment of binomial row `n` is
/// `n * (n - 1) * 2^(n - 2)`.
theorem binom_second_weighted_row_sum(n: Nat) {
    partial(binom_second_weighted_term(n), n.suc) =
        n * (n - Nat.1) * 2.pow((n - Nat.1) - Nat.1)
} by {
    if n = Nat.0 {
        partial_one[Nat](binom_second_weighted_term(Nat.0))
        binom_second_weighted_term(Nat.0, Nat.0) = Nat.0
        partial(binom_second_weighted_term(Nat.0), Nat.1) = Nat.0
        Nat.0 * (Nat.0 - Nat.1) * 2.pow((Nat.0 - Nat.1) - Nat.1) = Nat.0
        partial(binom_second_weighted_term(n), n.suc) =
            n * (n - Nat.1) * 2.pow((n - Nat.1) - Nat.1)
    } else {
        let m: Nat satisfy {
            m.suc = n
        }
        m = n - Nat.1
        partial_drop_first[Nat](binom_second_weighted_term(n), n.suc)
        binom_second_weighted_term(n, Nat.0) = Nat.0
        partial(binom_second_weighted_term(n), n.suc) =
            partial(compose(binom_second_weighted_term(n), Nat.suc), n)

        forall(k: Nat) {
            if k < n {
                k <= m
                binom_absorption_suc(m, k)
                k.suc * m.suc.binom(k.suc) = m.suc * m.binom(k)
                k.suc - Nat.1 = k
                binom_second_weighted_term(n, k.suc) =
                    k.suc * k * n.binom(k.suc)
                k.suc * k * n.binom(k.suc) =
                    k * (k.suc * n.binom(k.suc))
                k * (k.suc * n.binom(k.suc)) = k * (n * m.binom(k))
                k * (n * m.binom(k)) = (k * n) * m.binom(k)
                k * n = n * k
                (k * n) * m.binom(k) = (n * k) * m.binom(k)
                (n * k) * m.binom(k) = n * (k * m.binom(k))
                k * (n * m.binom(k)) = n * (k * m.binom(k))
                binom_weighted_term(m, k) = k * m.binom(k)
                binom_second_weighted_term(n, k.suc) = n * binom_weighted_term(m, k)
                compose(binom_second_weighted_term(n), Nat.suc, k) =
                    mul_fn(n, binom_weighted_term(m), k)
            }
        }
        partial_pointwise_eq[Nat](
            compose(binom_second_weighted_term(n), Nat.suc),
            mul_fn(n, binom_weighted_term(m)), n)
        partial(compose(binom_second_weighted_term(n), Nat.suc), n) =
            partial(mul_fn(n, binom_weighted_term(m)), n)
        partial_scalar_mul[Nat](n, binom_weighted_term(m), n)
        n * partial(binom_weighted_term(m), n) =
            partial(mul_fn(n, binom_weighted_term(m)), n)
        binom_weighted_row_sum(m)
        partial(binom_weighted_term(m), m.suc) = m * 2.pow(m - Nat.1)
        partial(binom_weighted_term(m), n) =
            (n - Nat.1) * 2.pow((n - Nat.1) - Nat.1)
        partial(binom_second_weighted_term(n), n.suc) =
            n * (n - Nat.1) * 2.pow((n - Nat.1) - Nat.1)
    }
    partial(binom_second_weighted_term(n), n.suc) =
        n * (n - Nat.1) * 2.pow((n - Nat.1) - Nat.1)
}

/// The `n`th triangular number: `0 + 1 + ... + n`.
define triangular(n: Nat) -> Nat {
    match n {
        Nat.zero {
            Nat.0
        }
        Nat.suc(k) {
            n + triangular(k)
        }
    }
}

/// Empty base case of `triangular`.
theorem triangular_zero {
    triangular(Nat.0) = Nat.0
}

/// Successor recurrence for `triangular`.
theorem triangular_suc(k: Nat) {
    triangular(k.suc) = k.suc + triangular(k)
}

/// Inductive step for `triangular_doubled`.
theorem triangular_doubled_suc(k: Nat) {
    Nat.2 * triangular(k) = k * (k + Nat.1)
        implies Nat.2 * triangular(k.suc) = k.suc * (k.suc + Nat.1)
} by {
    if Nat.2 * triangular(k) = k * (k + Nat.1) {
        triangular_suc(k)
        distrib_left(Nat.2, k.suc, triangular(k))
        distrib_right(k.suc, k, Nat.2)
        k + Nat.2 = k.suc.suc
        k.suc * k.suc.suc = k.suc * k + k.suc * Nat.2
        k.suc * k.suc.suc = Nat.2 * k.suc + k * (k + Nat.1)
        Nat.2 * triangular(k.suc) = k.suc * (k.suc + Nat.1)
    }
}

/// `2 * triangular(n) = n * (n + 1)`. The closed form, in `Nat` (no division).
theorem triangular_doubled(n: Nat) {
    Nat.2 * triangular(n) = n * (n + Nat.1)
} by {
    define p(x: Nat) -> Bool {
        Nat.2 * triangular(x) = x * (x + Nat.1)
    }
    triangular_zero
    p(Nat.0)
    forall(k: Nat) {
        if p(k) {
            triangular_doubled_suc(k)
            p(k.suc)
        }
    }
    forall(k: Nat) { p(k) implies p(k.suc) }
    p(Nat.0) and forall(k: Nat) { p(k) implies p(k.suc) }
    alt_induction(p)
    forall(k: Nat) { p(k) }
}
