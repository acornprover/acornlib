/// Generating functions over the reals.
///
/// A generating function is represented by its coefficient sequence
/// `a: Nat -> Real`, viewed as the formal power series `Σ a(n) x^n`.  Where
/// the series converges in the analytic sense developed in the real library,
/// the value of the generating function at `x` is
/// `limit(partial(seq_mul(a, x.pow)))`.
///
/// This file records the generating functions of the classic sequences —
/// the geometric series, the exponential series and the binomial series —
/// and proves the central identities: the Cauchy-product (convolution)
/// identity, the square of the geometric series, Vandermonde's identity,
/// and the recurrence that governs the binomial series `1/(1-x)^(k+1)`.

from nat import Nat, alt_induction, from_nat, from_nat_add, from_nat_mul, from_nat_zero,
    from_nat_one, pow_zero, pow_one, pow_add, factorial_zero,
    add_sub, add_imp_sub_left, sub_self, lte_suc_suc, lt_suc
from rat import Rat
from pair import Pair
from list import partial, partial_zero, partial_one, partial_split_last,
    partial_pointwise_eq, partial_scalar_mul, partial_reverse, reverse_index
from algebra.add_group import inverse_add, inverse_left
from algebra.semigroup import mul_fn
from data.basic.functions import compose
from real import Real, converges, converges_to, limit, geom_series, geom_converges,
    exp_term, cauchy_product, cauchy_coefficient, cauchy_seq, cauchy_product_converges,
    absolutely_converges, abs_fn, mul_abs, abs_not_neg, pos_imp_eq_abs, div_div,
    converges_to_has_tail_bound, tail_bound, tail_bound_implies_is_close
from combinatorics import binom, vandermonde, vandermonde_term, hockey_stick, hockey_stick_term, choose_n

numerals Real
numerals Nat

/// Two negated summands cancel against their sum: -b + -a + (b + a) = 0.
/// This is the algebraic core of the Fibonacci characteristic identity.
theorem neg_pair_cancel_ring(a: Real, b: Real) {
    (-b + -a) + (b + a) = Real.0
} by {
    inverse_add(b, a)
    -(b + a) = -a + -b
    -a + -b = -b + -a
    -(b + a) = -b + -a
    -b + -a = -(b + a)
    inverse_left(b + a)
    -(b + a) + (b + a) = Real.0
    (-b + -a) + (b + a) = Real.0
}

/// The constant-one sequence: every coefficient is 1.
define one_seq(n: Nat) -> Real {
    Real.1
}

/// The n-th term of the geometric series in x: x^n.
define geom_term(x: Real, n: Nat) -> Real {
    x.pow(n)
}

/// The pointwise product of two sequences.
define seq_mul(a: Nat -> Real, b: Nat -> Real, n: Nat) -> Real {
    a(n) * b(n)
}

/// The coefficient sequence of 1/(1-x)^2: n + 1.
define geom_square_coeff_seq(n: Nat) -> Real {
    from_nat[Real](n.suc)
}

/// The n-th coefficient of the binomial series 1/(1-x)^(k+1): C(n + k, k).
define binom_series_coeff(k: Nat, n: Nat) -> Real {
    from_nat[Real]((n + k).binom(k))
}

/// The n-th coefficient of (1 + x)^m, extended by zero beyond m.
define binom_row_coeff(m: Nat, n: Nat) -> Real {
    from_nat[Real](m.binom(n))
}

/// A pair (fib(n), fib(n+1)) used to define the Fibonacci numbers iteratively.
define fib_pair(n: Nat) -> Pair[Nat, Nat] {
    match n {
        Nat.zero {
            Pair.new(Nat.0, Nat.1)
        }
        Nat.suc(k) {
            let p: Pair[Nat, Nat] = fib_pair(k)
            Pair.new(p.second, p.first + p.second)
        }
    }
}

/// The n-th Fibonacci number, with fib(0) = 0 and fib(1) = 1.
define fib(n: Nat) -> Nat {
    fib_pair(n).first
}

/// The Fibonacci numbers as a real sequence.
define fib_seq(n: Nat) -> Real {
    from_nat[Real](fib(n))
}

/// The successor step of the Fibonacci pair iteration.
theorem fib_pair_suc(k: Nat) {
    fib_pair(k.suc) = Pair.new(fib_pair(k).second, fib_pair(k).first + fib_pair(k).second)
}

/// The base value of the Fibonacci pair iteration.
theorem fib_pair_zero {
    fib_pair(Nat.0) = Pair.new(Nat.0, Nat.1)
}

/// The second component of the Fibonacci pair is the next Fibonacci number.
theorem fib_pair_second_suc(k: Nat) {
    fib_pair(k).second = fib(k.suc)
} by {
    fib(k.suc) = fib_pair(k.suc).first
    fib_pair_suc(k)
    fib_pair(k.suc).first = Pair.new(fib_pair(k).second, fib_pair(k).first + fib_pair(k).second).first
    Pair.new(fib_pair(k).second, fib_pair(k).first + fib_pair(k).second).first = fib_pair(k).second
    fib_pair(k).second = fib(k.suc)
}

/// The zero-th Fibonacci number is zero.
theorem fib_zero {
    fib(Nat.0) = Nat.0
}

/// The first Fibonacci number is one.
theorem fib_one {
    fib(Nat.1) = Nat.1
} by {
    fib(Nat.1) = fib_pair(Nat.1).first
    fib_pair_suc(Nat.0)
    fib_pair(Nat.1).first = Pair.new(fib_pair(Nat.0).second, fib_pair(Nat.0).first + fib_pair(Nat.0).second).first
    Pair.new(fib_pair(Nat.0).second, fib_pair(Nat.0).first + fib_pair(Nat.0).second).first = fib_pair(Nat.0).second
    fib_pair(Nat.0).second = Pair.new(Nat.0, Nat.1).second
    Pair.new(Nat.0, Nat.1).second = Nat.1
    fib(Nat.1) = Nat.1
}

/// The Fibonacci recurrence: fib(n + 2) = fib(n + 1) + fib(n).
theorem fib_suc_suc(j: Nat) {
    fib(j.suc.suc) = fib(j.suc) + fib(j)
} by {
    fib(j.suc.suc) = fib_pair(j.suc.suc).first
    fib_pair_suc(j.suc)
    fib_pair(j.suc.suc).first = Pair.new(fib_pair(j.suc).second, fib_pair(j.suc).first + fib_pair(j.suc).second).first
    Pair.new(fib_pair(j.suc).second, fib_pair(j.suc).first + fib_pair(j.suc).second).first = fib_pair(j.suc).second
    fib_pair_second_suc(j)
    fib_pair(j).second = fib(j.suc)
    fib_pair_suc(j)
    fib_pair(j.suc).second = Pair.new(fib_pair(j).second, fib_pair(j).first + fib_pair(j).second).second
    Pair.new(fib_pair(j).second, fib_pair(j).first + fib_pair(j).second).second = fib_pair(j).first + fib_pair(j).second
    fib_pair(j).first = fib(j)
    fib(j.suc.suc) = fib(j) + fib(j.suc)
    fib(j) + fib(j.suc) = fib(j.suc) + fib(j)
    fib(j.suc.suc) = fib(j.suc) + fib(j)
}

/// The embedding of natural numbers into reals commutes with partial sums.
theorem from_nat_partial(f: Nat -> Nat, n: Nat) {
    from_nat[Real](partial(f, n)) = partial(compose(from_nat[Real], f), n)
} by {
    define p(m: Nat) -> Bool {
        from_nat[Real](partial(f, m)) = partial(compose(from_nat[Real], f), m)
    }
    partial_zero(f)
    partial(f, Nat.0) = Nat.0
    from_nat_zero[Real]
    from_nat[Real](Nat.0) = Real.0
    partial_zero(compose(from_nat[Real], f))
    partial(compose(from_nat[Real], f), Nat.0) = Real.0
    p(Nat.0)
    forall(m: Nat) {
        if p(m) {
            partial_split_last(f, m)
            partial(f, m.suc) = partial(f, m) + f(m)
            from_nat_add[Real](partial(f, m), f(m))
            from_nat[Real](partial(f, m) + f(m)) =
                from_nat[Real](partial(f, m)) + from_nat[Real](f(m))
            from_nat[Real](partial(f, m.suc)) =
                from_nat[Real](partial(f, m)) + from_nat[Real](f(m))
            from_nat[Real](partial(f, m)) = partial(compose(from_nat[Real], f), m)
            from_nat[Real](partial(f, m.suc)) =
                partial(compose(from_nat[Real], f), m) + from_nat[Real](f(m))
            partial_split_last(compose(from_nat[Real], f), m)
            partial(compose(from_nat[Real], f), m.suc) =
                partial(compose(from_nat[Real], f), m) + compose(from_nat[Real], f, m)
            compose(from_nat[Real], f, m) = from_nat[Real](f(m))
            partial(compose(from_nat[Real], f), m.suc) =
                partial(compose(from_nat[Real], f), m) + from_nat[Real](f(m))
            from_nat[Real](partial(f, m.suc)) =
                partial(compose(from_nat[Real], f), m.suc)
            p(m.suc)
        }
    }
    p(Nat.0) and forall(m: Nat) { p(m) implies p(m.suc) }
    alt_induction(p)
    forall(m: Nat) { p(m) }
    p(n)
}

/// The partial sums of the constant-one sequence are the counting function.
theorem one_seq_partial(m: Nat) {
    partial(one_seq, m) = from_nat[Real](m)
} by {
    define p(k: Nat) -> Bool {
        partial(one_seq, k) = from_nat[Real](k)
    }
    partial_zero(one_seq)
    partial(one_seq, Nat.0) = Real.0
    from_nat_zero[Real]
    from_nat[Real](Nat.0) = Real.0
    p(Nat.0)
    forall(k: Nat) {
        if p(k) {
            partial_split_last(one_seq, k)
            partial(one_seq, k.suc) = partial(one_seq, k) + one_seq(k)
            partial(one_seq, k) = from_nat[Real](k)
            one_seq(k) = Real.1
            partial(one_seq, k.suc) = from_nat[Real](k) + Real.1
            from_nat_add[Real](k, Nat.1)
            from_nat[Real](k + Nat.1) = from_nat[Real](k) + from_nat[Real](Nat.1)
            k + Nat.1 = k.suc
            from_nat[Real](k.suc) = from_nat[Real](k) + from_nat[Real](Nat.1)
            from_nat_one[Real]
            from_nat[Real](Nat.1) = Real.1
            from_nat[Real](k.suc) = from_nat[Real](k) + Real.1
            partial(one_seq, k.suc) = from_nat[Real](k.suc)
            p(k.suc)
        }
    }
    p(Nat.0) and forall(k: Nat) { p(k) implies p(k.suc) }
    alt_induction(p)
    forall(k: Nat) { p(k) }
    p(m)
}

// ---------------------------------------------------------------------------
// The geometric series: Σ x^n = 1/(1-x).
// ---------------------------------------------------------------------------

/// The geometric series: the generating function of the geometric term
/// sequence `x^n` is 1/(1-x) whenever |x| < 1.
/// This restates the library theorem `geom_series`.
theorem geom_gf(x: Real) {
    x.abs < Real.1 implies
    limit(partial(geom_term(x))) = Real.1 / (Real.1 - x)
} by {
    geom_series(x)
    limit(partial(x.pow)) = Real.1 / (Real.1 - x)
    forall(n: Nat) {
        geom_term(x, n) = x.pow(n)
    }
    partial(geom_term(x)) = partial(x.pow)
    limit(partial(geom_term(x))) = limit(partial(x.pow))
    limit(partial(geom_term(x))) = Real.1 / (Real.1 - x)
}

/// The generating function of the constant-one sequence is 1/(1-x).
theorem one_seq_gf(x: Real) {
    x.abs < Real.1 implies
    limit(partial(seq_mul(one_seq, x.pow))) = Real.1 / (Real.1 - x)
} by {
    forall(n: Nat) {
        seq_mul(one_seq, x.pow, n) = one_seq(n) * x.pow(n)
        one_seq(n) = Real.1
        seq_mul(one_seq, x.pow, n) = Real.1 * x.pow(n)
        Real.1 * x.pow(n) = x.pow(n)
        seq_mul(one_seq, x.pow, n) = x.pow(n)
    }
    partial(seq_mul(one_seq, x.pow)) = partial(x.pow)
    geom_series(x)
    limit(partial(x.pow)) = Real.1 / (Real.1 - x)
    limit(partial(seq_mul(one_seq, x.pow))) = limit(partial(x.pow))
    limit(partial(seq_mul(one_seq, x.pow))) = Real.1 / (Real.1 - x)
}

// ---------------------------------------------------------------------------
// The exponential series: e^x = Σ x^n / n!.
// ---------------------------------------------------------------------------

/// The exponential function is the generating function of the sequence
/// 1/n!: e^x = Σ_{n>=0} x^n / n!.  This is the definition of Real.exp.
theorem exp_gf(x: Real) {
    x.exp = limit(partial(exp_term(x)))
}

/// The first coefficient of the exponential series is 1.
theorem exp_gf_zero_coeff(x: Real) {
    exp_term(x, Nat.0) = Real.1
} by {
    exp_term(x, Nat.0) = x.pow(Nat.0) / Real.from_rat(Rat.from_nat(Nat.0.factorial))
    factorial_zero
    Nat.0.factorial = Nat.1
    x.pow(Nat.0) = Real.1
    Rat.from_nat(Nat.0.factorial) = Rat.1
    Real.from_rat(Rat.1) = Real.1
    Real.1 / Real.1 = Real.1
    exp_term(x, Nat.0) = Real.1
}

/// The coefficient of x in the exponential series is x itself.
theorem exp_gf_one_coeff(x: Real) {
    exp_term(x, Nat.1) = x
} by {
    exp_term(x, Nat.1) = x.pow(Nat.1) / Real.from_rat(Rat.from_nat(Nat.1.factorial))
    Nat.1.factorial = Nat.1
    x.pow(Nat.1) = x
    Rat.from_nat(Nat.1.factorial) = Rat.1
    Real.from_rat(Rat.1) = Real.1
    x / Real.1 = x
    exp_term(x, Nat.1) = x
}

// ---------------------------------------------------------------------------
// The convolution identity (Cauchy product).
// ---------------------------------------------------------------------------

/// The convolution identity: the n-th coefficient of the product of two
/// generating functions is the convolution of their coefficient sequences,
/// Σ_{k=0}^n a(k) * b(n-k).
theorem convolve_coeff_sum(a: Nat -> Real, b: Nat -> Real, n: Nat) {
    cauchy_product(a, b, n) = partial(cauchy_coefficient(a, b, n), n.suc)
}

/// Convolving with the constant-one sequence computes running sums:
/// Σ_{k=0}^n a(k) is the n-th coefficient of (Σ x^n)(Σ a(n) x^n).
theorem convolve_one(a: Nat -> Real, n: Nat) {
    cauchy_product(one_seq, a, n) = partial(a, n.suc)
} by {
    convolve_coeff_sum(one_seq, a, n)
    cauchy_product(one_seq, a, n) = partial(cauchy_coefficient(one_seq, a, n), n.suc)
    forall(k: Nat) {
        if k < n.suc {
            cauchy_coefficient(one_seq, a, n, k) = one_seq(k) * a(n - k)
            one_seq(k) = Real.1
            cauchy_coefficient(one_seq, a, n, k) = Real.1 * a(n - k)
            Real.1 * a(n - k) = a(n - k)
            reverse_index(a, n, k) = a(n - k)
            cauchy_coefficient(one_seq, a, n, k) = reverse_index(a, n, k)
        }
    }
    partial_pointwise_eq[Real](cauchy_coefficient(one_seq, a, n), reverse_index(a, n), n.suc)
    partial(cauchy_coefficient(one_seq, a, n), n.suc) = partial(reverse_index(a, n), n.suc)
    partial_reverse[Real](a, n)
    partial(a, n.suc) = partial(reverse_index(a, n), n.suc)
    partial(reverse_index(a, n), n.suc) = partial(a, n.suc)
    partial(cauchy_coefficient(one_seq, a, n), n.suc) = partial(a, n.suc)
    cauchy_product(one_seq, a, n) = partial(a, n.suc)
}

/// Multiplying both coefficient sequences by x^n scales the n-th convolution
/// coefficient by x^n: the coefficient of x^n in the product of
/// (Σ a(k) x^k)(Σ b(j) x^j) is (Σ_{k=0}^n a(k) b(n-k)) x^n.
theorem cauchy_scaled_coeff(a: Nat -> Real, b: Nat -> Real, x: Real, n: Nat) {
    cauchy_seq(seq_mul(a, x.pow), seq_mul(b, x.pow))(n) =
        cauchy_product(a, b, n) * x.pow(n)
} by {
    cauchy_seq(seq_mul(a, x.pow), seq_mul(b, x.pow))(n) =
        cauchy_product(seq_mul(a, x.pow), seq_mul(b, x.pow), n)
    convolve_coeff_sum(seq_mul(a, x.pow), seq_mul(b, x.pow), n)
    cauchy_product(seq_mul(a, x.pow), seq_mul(b, x.pow), n) =
        partial(cauchy_coefficient(seq_mul(a, x.pow), seq_mul(b, x.pow), n), n.suc)
    forall(k: Nat) {
        if k < n.suc {
            k <= n
            cauchy_coefficient(seq_mul(a, x.pow), seq_mul(b, x.pow), n, k) =
                seq_mul(a, x.pow, k) * seq_mul(b, x.pow, n - k)
            seq_mul(a, x.pow, k) = a(k) * x.pow(k)
            seq_mul(b, x.pow, n - k) = b(n - k) * x.pow(n - k)
            seq_mul(a, x.pow, k) * seq_mul(b, x.pow, n - k) =
                (a(k) * x.pow(k)) * (b(n - k) * x.pow(n - k))
            (a(k) * x.pow(k)) * (b(n - k) * x.pow(n - k)) =
                a(k) * (x.pow(k) * (b(n - k) * x.pow(n - k)))
            x.pow(k) * (b(n - k) * x.pow(n - k)) =
                (x.pow(k) * b(n - k)) * x.pow(n - k)
            x.pow(k) * b(n - k) = b(n - k) * x.pow(k)
            (x.pow(k) * b(n - k)) * x.pow(n - k) =
                (b(n - k) * x.pow(k)) * x.pow(n - k)
            a(k) * ((b(n - k) * x.pow(k)) * x.pow(n - k)) =
                a(k) * (b(n - k) * (x.pow(k) * x.pow(n - k)))
            a(k) * (b(n - k) * (x.pow(k) * x.pow(n - k))) =
                (a(k) * b(n - k)) * (x.pow(k) * x.pow(n - k))
            (a(k) * x.pow(k)) * (b(n - k) * x.pow(n - k)) =
                (a(k) * b(n - k)) * (x.pow(k) * x.pow(n - k))
            seq_mul(a, x.pow, k) * seq_mul(b, x.pow, n - k) =
                (a(k) * b(n - k)) * (x.pow(k) * x.pow(n - k))
            pow_add(x, k, n - k)
            x.pow(k) * x.pow(n - k) = x.pow(k + (n - k))
            k + (n - k) = n
            x.pow(k) * x.pow(n - k) = x.pow(n)
            (a(k) * b(n - k)) * (x.pow(k) * x.pow(n - k)) =
                (a(k) * b(n - k)) * x.pow(n)
            cauchy_coefficient(seq_mul(a, x.pow), seq_mul(b, x.pow), n, k) =
                (a(k) * b(n - k)) * x.pow(n)
            cauchy_coefficient(a, b, n, k) = a(k) * b(n - k)
            (a(k) * b(n - k)) * x.pow(n) = cauchy_coefficient(a, b, n, k) * x.pow(n)
            mul_fn(x.pow(n), cauchy_coefficient(a, b, n), k) =
                x.pow(n) * cauchy_coefficient(a, b, n, k)
            cauchy_coefficient(a, b, n, k) * x.pow(n) =
                mul_fn(x.pow(n), cauchy_coefficient(a, b, n), k)
            cauchy_coefficient(seq_mul(a, x.pow), seq_mul(b, x.pow), n, k) =
                mul_fn(x.pow(n), cauchy_coefficient(a, b, n), k)
        }
    }
    partial_pointwise_eq[Real](
        cauchy_coefficient(seq_mul(a, x.pow), seq_mul(b, x.pow), n),
        mul_fn(x.pow(n), cauchy_coefficient(a, b, n)), n.suc)
    partial(cauchy_coefficient(seq_mul(a, x.pow), seq_mul(b, x.pow), n), n.suc) =
        partial(mul_fn(x.pow(n), cauchy_coefficient(a, b, n)), n.suc)
    partial_scalar_mul[Real](x.pow(n), cauchy_coefficient(a, b, n), n.suc)
    x.pow(n) * partial(cauchy_coefficient(a, b, n), n.suc) =
        partial(mul_fn(x.pow(n), cauchy_coefficient(a, b, n)), n.suc)
    partial(mul_fn(x.pow(n), cauchy_coefficient(a, b, n)), n.suc) =
        x.pow(n) * partial(cauchy_coefficient(a, b, n), n.suc)
    convolve_coeff_sum(a, b, n)
    cauchy_product(a, b, n) = partial(cauchy_coefficient(a, b, n), n.suc)
    partial(cauchy_coefficient(a, b, n), n.suc) = cauchy_product(a, b, n)
    x.pow(n) * partial(cauchy_coefficient(a, b, n), n.suc) =
        cauchy_product(a, b, n) * x.pow(n)
    partial(cauchy_coefficient(seq_mul(a, x.pow), seq_mul(b, x.pow), n), n.suc) =
        cauchy_product(a, b, n) * x.pow(n)
    cauchy_product(seq_mul(a, x.pow), seq_mul(b, x.pow), n) =
        cauchy_product(a, b, n) * x.pow(n)
    cauchy_seq(seq_mul(a, x.pow), seq_mul(b, x.pow))(n) =
        cauchy_product(a, b, n) * x.pow(n)
}

/// Converges-to respects pointwise equality of sequences.
theorem converges_to_eq(f: Nat -> Real, g: Nat -> Real, l: Real) {
    (forall(n: Nat) { f(n) = g(n) })
    implies (converges_to(f, l) implies converges_to(g, l))
} by {
    if forall(n: Nat) { f(n) = g(n) } {
        if converges_to(f, l) {
            forall(eps: Real) {
                if eps.is_positive {
                    converges_to_has_tail_bound(f, l, eps)
                    converges_to(f, l) and eps.is_positive
                    exists(n0: Nat) {
                        tail_bound(f, l, n0, eps)
                    }
                    let n0: Nat satisfy {
                        tail_bound(f, l, n0, eps)
                    }
                    forall(i: Nat) {
                        if n0 <= i {
                            tail_bound_implies_is_close(f, l, n0, eps, i)
                            tail_bound(f, l, n0, eps) and n0 <= i
                            f(i).is_close(l, eps)
                            f(i) = g(i)
                            g(i).is_close(l, eps)
                        }
                    }
                    tail_bound(g, l, n0, eps)
                    exists(n1: Nat) {
                        tail_bound(g, l, n1, eps)
                    }
                }
            }
            converges_to(g, l)
        }
    }
}

/// The convolution identity for generating functions: if both coefficient
/// series converge absolutely at x, the series of convolution coefficients
/// converges to the product of the two generating-function values.
theorem gf_product_limit(a: Nat -> Real, b: Nat -> Real, x: Real) {
    absolutely_converges(seq_mul(a, x.pow)) and absolutely_converges(seq_mul(b, x.pow))
    implies
    converges_to(partial(cauchy_seq(seq_mul(a, x.pow), seq_mul(b, x.pow))),
        limit(partial(seq_mul(a, x.pow))) * limit(partial(seq_mul(b, x.pow))))
} by {
    cauchy_product_converges(seq_mul(a, x.pow), seq_mul(b, x.pow))
    converges_to(partial(cauchy_seq(seq_mul(a, x.pow), seq_mul(b, x.pow))),
        limit(partial(seq_mul(a, x.pow))) * limit(partial(seq_mul(b, x.pow))))
}

// ---------------------------------------------------------------------------
// The square of the geometric series: 1/(1-x)^2 = Σ (n+1) x^n.
// ---------------------------------------------------------------------------

/// The coefficient of x^n in 1/(1-x)^2 is n+1: the square of the geometric
/// series is the convolution of the constant-one sequence with itself.
theorem geom_square_coeff(n: Nat) {
    cauchy_product(one_seq, one_seq, n) = from_nat[Real](n.suc)
} by {
    convolve_coeff_sum(one_seq, one_seq, n)
    cauchy_product(one_seq, one_seq, n) = partial(cauchy_coefficient(one_seq, one_seq, n), n.suc)
    forall(k: Nat) {
        if k < n.suc {
            cauchy_coefficient(one_seq, one_seq, n, k) = one_seq(k) * one_seq(n - k)
            one_seq(k) = Real.1
            one_seq(n - k) = Real.1
            one_seq(k) * one_seq(n - k) = Real.1
            cauchy_coefficient(one_seq, one_seq, n, k) = one_seq(k)
        }
    }
    partial_pointwise_eq[Real](cauchy_coefficient(one_seq, one_seq, n), one_seq, n.suc)
    partial(cauchy_coefficient(one_seq, one_seq, n), n.suc) = partial(one_seq, n.suc)
    one_seq_partial(n.suc)
    partial(one_seq, n.suc) = from_nat[Real](n.suc)
    partial(cauchy_coefficient(one_seq, one_seq, n), n.suc) = from_nat[Real](n.suc)
    cauchy_product(one_seq, one_seq, n) = from_nat[Real](n.suc)
}

// ---------------------------------------------------------------------------
// The binomial series: 1/(1-x)^(k+1) = Σ C(n+k, k) x^n.
// ---------------------------------------------------------------------------

/// The constant term of the binomial series 1/(1-x)^(k+1) is 1.
theorem binom_series_zero(k: Nat) {
    binom_series_coeff(k, Nat.0) = Real.1
} by {
    binom_series_coeff(k, Nat.0) = from_nat[Real]((Nat.0 + k).binom(k))
    Nat.0 + k = k
    binom_series_coeff(k, Nat.0) = from_nat[Real](k.binom(k))
    choose_n(k)
    k.binom(k) = Nat.1
    from_nat[Real](k.binom(k)) = from_nat[Real](Nat.1)
    from_nat_one[Real]
    from_nat[Real](Nat.1) = Real.1
    binom_series_coeff(k, Nat.0) = Real.1
}

/// The binomial series obeys the hockey-stick recurrence: the coefficient
/// sequence for the exponent k+1 is the running sum of the coefficient
/// sequence for the exponent k.  This is the formal counterpart of
/// multiplying 1/(1-x)^(k+1) by 1/(1-x).
theorem binom_series_sums(k: Nat, m: Nat) {
    partial(binom_series_coeff(k), m.suc) = binom_series_coeff(k.suc, m)
} by {
    forall(n: Nat) {
        if n < m.suc {
            binom_series_coeff(k, n) = from_nat[Real]((n + k).binom(k))
            n + k = k + n
            binom_series_coeff(k, n) = from_nat[Real]((k + n).binom(k))
            hockey_stick_term(k, n) = (k + n).binom(k)
            from_nat[Real]((k + n).binom(k)) = from_nat[Real](hockey_stick_term(k, n))
            binom_series_coeff(k, n) = compose(from_nat[Real], hockey_stick_term(k), n)
        }
    }
    partial_pointwise_eq[Real](binom_series_coeff(k), compose(from_nat[Real], hockey_stick_term(k)), m.suc)
    partial(binom_series_coeff(k), m.suc) = partial(compose(from_nat[Real], hockey_stick_term(k)), m.suc)
    from_nat_partial(hockey_stick_term(k), m.suc)
    from_nat[Real](partial(hockey_stick_term(k), m.suc)) =
        partial(compose(from_nat[Real], hockey_stick_term(k)), m.suc)
    partial(compose(from_nat[Real], hockey_stick_term(k)), m.suc) =
        from_nat[Real](partial(hockey_stick_term(k), m.suc))
    partial(binom_series_coeff(k), m.suc) =
        from_nat[Real](partial(hockey_stick_term(k), m.suc))
    hockey_stick(k, m)
    partial(hockey_stick_term(k), m.suc) = (k + m.suc).binom(k.suc)
    from_nat[Real](partial(hockey_stick_term(k), m.suc)) =
        from_nat[Real]((k + m.suc).binom(k.suc))
    k + m.suc = m.suc + k
    m.suc + k = m + k.suc
    k + m.suc = m + k.suc
    from_nat[Real]((k + m.suc).binom(k.suc)) =
        from_nat[Real]((m + k.suc).binom(k.suc))
    partial(binom_series_coeff(k), m.suc) =
        from_nat[Real]((m + k.suc).binom(k.suc))
    binom_series_coeff(k.suc, m) = from_nat[Real]((m + k.suc).binom(k.suc))
    partial(binom_series_coeff(k), m.suc) = binom_series_coeff(k.suc, m)
}

// ---------------------------------------------------------------------------
// Vandermonde's identity via generating functions.
// ---------------------------------------------------------------------------

/// Vandermonde's identity: the coefficient of x^k in (1+x)^m (1+x)^n is
/// C(m+n, k).  In generating-function language, the convolution of the
/// coefficient sequences of (1+x)^m and (1+x)^n is the coefficient
/// sequence of (1+x)^(m+n).
theorem vandermonde_gf(m: Nat, n: Nat, k: Nat) {
    cauchy_product(binom_row_coeff(m), binom_row_coeff(n), k) =
        from_nat[Real]((m + n).binom(k))
} by {
    convolve_coeff_sum(binom_row_coeff(m), binom_row_coeff(n), k)
    cauchy_product(binom_row_coeff(m), binom_row_coeff(n), k) =
        partial(cauchy_coefficient(binom_row_coeff(m), binom_row_coeff(n), k), k.suc)
    forall(j: Nat) {
        if j < k.suc {
            cauchy_coefficient(binom_row_coeff(m), binom_row_coeff(n), k, j) =
                binom_row_coeff(m, j) * binom_row_coeff(n, k - j)
            binom_row_coeff(m, j) = from_nat[Real](m.binom(j))
            binom_row_coeff(n, k - j) = from_nat[Real](n.binom(k - j))
            binom_row_coeff(m, j) * binom_row_coeff(n, k - j) =
                from_nat[Real](m.binom(j)) * from_nat[Real](n.binom(k - j))
            from_nat_mul[Real](m.binom(j), n.binom(k - j))
            from_nat[Real](m.binom(j) * n.binom(k - j)) =
                from_nat[Real](m.binom(j)) * from_nat[Real](n.binom(k - j))
            from_nat[Real](m.binom(j)) * from_nat[Real](n.binom(k - j)) =
                from_nat[Real](m.binom(j) * n.binom(k - j))
            binom_row_coeff(m, j) * binom_row_coeff(n, k - j) =
                from_nat[Real](m.binom(j) * n.binom(k - j))
            vandermonde_term(m, n, k, j) = m.binom(j) * n.binom(k - j)
            from_nat[Real](m.binom(j) * n.binom(k - j)) =
                from_nat[Real](vandermonde_term(m, n, k, j))
            binom_row_coeff(m, j) * binom_row_coeff(n, k - j) =
                from_nat[Real](vandermonde_term(m, n, k, j))
            cauchy_coefficient(binom_row_coeff(m), binom_row_coeff(n), k, j) =
                compose(from_nat[Real], vandermonde_term(m, n, k), j)
        }
    }
    partial_pointwise_eq[Real](
        cauchy_coefficient(binom_row_coeff(m), binom_row_coeff(n), k),
        compose(from_nat[Real], vandermonde_term(m, n, k)), k.suc)
    partial(cauchy_coefficient(binom_row_coeff(m), binom_row_coeff(n), k), k.suc) =
        partial(compose(from_nat[Real], vandermonde_term(m, n, k)), k.suc)
    from_nat_partial(vandermonde_term(m, n, k), k.suc)
    from_nat[Real](partial(vandermonde_term(m, n, k), k.suc)) =
        partial(compose(from_nat[Real], vandermonde_term(m, n, k)), k.suc)
    partial(compose(from_nat[Real], vandermonde_term(m, n, k)), k.suc) =
        from_nat[Real](partial(vandermonde_term(m, n, k), k.suc))
    partial(cauchy_coefficient(binom_row_coeff(m), binom_row_coeff(n), k), k.suc) =
        from_nat[Real](partial(vandermonde_term(m, n, k), k.suc))
    cauchy_product(binom_row_coeff(m), binom_row_coeff(n), k) =
        from_nat[Real](partial(vandermonde_term(m, n, k), k.suc))
    vandermonde(m, n, k)
    partial(vandermonde_term(m, n, k), k.suc) = (m + n).binom(k)
    from_nat[Real](partial(vandermonde_term(m, n, k), k.suc)) =
        from_nat[Real]((m + n).binom(k))
    cauchy_product(binom_row_coeff(m), binom_row_coeff(n), k) =
        from_nat[Real]((m + n).binom(k))
}

// ---------------------------------------------------------------------------
// Analytic identities: the value of the generating function at x.
// ---------------------------------------------------------------------------

/// The absolute value of one is one.
theorem one_abs {
    Real.1.abs = Real.1
} by {
    Real.1.is_positive
    if Real.1.is_negative {
        false
    }
    if not Real.1.is_negative {
        Real.1.abs = Real.1
    }
}

/// The absolute value of the absolute value is the absolute value.
theorem abs_idem(x: Real) {
    x.abs.abs = x.abs
} by {
    abs_not_neg(x)
    if x.abs.is_negative {
        false
    }
    if not x.abs.is_negative {
        x.abs.abs = x.abs
    }
}

/// The absolute value of a power is the power of the absolute value.
theorem pow_abs_seq(x: Real, n: Nat) {
    x.pow(n).abs = x.abs.pow(n)
} by {
    define p(k: Nat) -> Bool {
        x.pow(k).abs = x.abs.pow(k)
    }
    pow_zero(x)
    x.pow(Nat.0) = Real.1
    x.pow(Nat.0).abs = Real.1.abs
    one_abs
    Real.1.abs = Real.1
    x.pow(Nat.0).abs = Real.1
    pow_zero(x.abs)
    x.abs.pow(Nat.0) = Real.1
    p(Nat.0)
    forall(k: Nat) {
        if p(k) {
            pow_add(x, k, Nat.1)
            x.pow(k) * x.pow(Nat.1) = x.pow(k + Nat.1)
            k + Nat.1 = k.suc
            x.pow(k) * x.pow(Nat.1) = x.pow(k.suc)
            pow_one(x)
            x.pow(Nat.1) = x
            x.pow(k) * x = x.pow(k.suc)
            x * x.pow(k) = x.pow(k.suc)
            x.pow(k.suc).abs = (x * x.pow(k)).abs
            mul_abs(x, x.pow(k))
            (x * x.pow(k)).abs = x.abs * x.pow(k).abs
            x.pow(k.suc).abs = x.abs * x.pow(k).abs
            x.pow(k).abs = x.abs.pow(k)
            x.pow(k.suc).abs = x.abs * x.abs.pow(k)
            pow_add(x.abs, k, Nat.1)
            x.abs.pow(k) * x.abs.pow(Nat.1) = x.abs.pow(k + Nat.1)
            k + Nat.1 = k.suc
            x.abs.pow(k) * x.abs.pow(Nat.1) = x.abs.pow(k.suc)
            pow_one(x.abs)
            x.abs.pow(Nat.1) = x.abs
            x.abs.pow(k) * x.abs = x.abs.pow(k.suc)
            x.abs * x.abs.pow(k) = x.abs.pow(k.suc)
            x.pow(k.suc).abs = x.abs.pow(k.suc)
            p(k.suc)
        }
    }
    p(Nat.0) and forall(k: Nat) { p(k) implies p(k.suc) }
    alt_induction(p)
    forall(k: Nat) { p(k) }
    p(n)
}

/// The geometric series converges absolutely whenever |x| < 1.
theorem geom_abs_conv(x: Real) {
    x.abs < Real.1 implies absolutely_converges(x.pow)
} by {
    if x.abs < Real.1 {
        forall(n: Nat) {
            abs_fn(x.pow, n) = x.pow(n).abs
            pow_abs_seq(x, n)
            x.pow(n).abs = x.abs.pow(n)
            abs_fn(x.pow, n) = x.abs.pow(n)
        }
        partial(abs_fn(x.pow)) = partial(x.abs.pow)
        abs_idem(x)
        x.abs.abs = x.abs
        x.abs.abs < Real.1
        geom_converges(x.abs)
        x.abs.abs < Real.1 implies converges(partial(x.abs.pow))
        converges(partial(x.abs.pow))
        converges(partial(abs_fn(x.pow)))
        absolutely_converges(x.pow) = converges(partial(abs_fn(x.pow)))
        absolutely_converges(x.pow)
    }
}

/// When |x| < 1, the denominator 1 - x of the geometric series is nonzero.
theorem one_sub_ne_zero(x: Real) {
    x.abs < Real.1 implies Real.1 - x != Real.0
} by {
    if x.abs < Real.1 {
        if Real.1 - x = Real.0 {
            Real.1 = x
            x.abs = Real.1.abs
            one_abs
            Real.1.abs = Real.1
            x.abs = Real.1
            x.abs < Real.1 and x.abs = Real.1
            false
        }
        Real.1 - x != Real.0
    }
}

/// The generating function of the sequence n + 1 is 1/(1-x)^2:
/// Σ_{n>=0} (n+1) x^n = 1/(1-x)^2 for |x| < 1.
theorem geom_square_limit(x: Real) {
    x.abs < Real.1 implies
    converges_to(partial(seq_mul(geom_square_coeff_seq, x.pow)),
        Real.1 / ((Real.1 - x) * (Real.1 - x)))
} by {
    if x.abs < Real.1 {
        geom_abs_conv(x)
        absolutely_converges(x.pow)
        cauchy_product_converges(x.pow, x.pow)
        absolutely_converges(x.pow) and absolutely_converges(x.pow)
        converges_to(partial(cauchy_seq(x.pow, x.pow)),
            limit(partial(x.pow)) * limit(partial(x.pow)))
        forall(m: Nat) {
            seq_mul(one_seq, x.pow, m) = one_seq(m) * x.pow(m)
            one_seq(m) = Real.1
            seq_mul(one_seq, x.pow, m) = Real.1 * x.pow(m)
            Real.1 * x.pow(m) = x.pow(m)
            seq_mul(one_seq, x.pow, m) = x.pow(m)
        }
        seq_mul(one_seq, x.pow) = x.pow
        forall(n: Nat) {
            cauchy_scaled_coeff(one_seq, one_seq, x, n)
            cauchy_seq(seq_mul(one_seq, x.pow), seq_mul(one_seq, x.pow))(n) =
                cauchy_product(one_seq, one_seq, n) * x.pow(n)
            cauchy_seq(x.pow, x.pow)(n) =
                cauchy_seq(seq_mul(one_seq, x.pow), seq_mul(one_seq, x.pow))(n)
            cauchy_seq(x.pow, x.pow)(n) = cauchy_product(one_seq, one_seq, n) * x.pow(n)
            geom_square_coeff(n)
            cauchy_product(one_seq, one_seq, n) = from_nat[Real](n.suc)
            cauchy_seq(x.pow, x.pow)(n) = from_nat[Real](n.suc) * x.pow(n)
            geom_square_coeff_seq(n) = from_nat[Real](n.suc)
            seq_mul(geom_square_coeff_seq, x.pow, n) =
                geom_square_coeff_seq(n) * x.pow(n)
            cauchy_seq(x.pow, x.pow)(n) = seq_mul(geom_square_coeff_seq, x.pow, n)
        }
        forall(n: Nat) {
            partial(cauchy_seq(x.pow, x.pow))(n) =
                partial(seq_mul(geom_square_coeff_seq, x.pow))(n)
        }
        converges_to_eq(partial(cauchy_seq(x.pow, x.pow)),
            partial(seq_mul(geom_square_coeff_seq, x.pow)),
            limit(partial(x.pow)) * limit(partial(x.pow)))
        converges_to(partial(seq_mul(geom_square_coeff_seq, x.pow)),
            limit(partial(x.pow)) * limit(partial(x.pow)))
        geom_series(x)
        limit(partial(x.pow)) = Real.1 / (Real.1 - x)
        limit(partial(x.pow)) * limit(partial(x.pow)) =
            (Real.1 / (Real.1 - x)) * (Real.1 / (Real.1 - x))
        one_sub_ne_zero(x)
        Real.1 - x != Real.0
        div_div(Real.1, Real.1 - x, Real.1, Real.1 - x)
        Real.1 - x != Real.0 and Real.1 != Real.0 and Real.1 - x != Real.0
        (Real.1 / (Real.1 - x)) * (Real.1 / (Real.1 - x)) =
            (Real.1 * Real.1) / ((Real.1 - x) * (Real.1 - x))
        Real.1 * Real.1 = Real.1
        (Real.1 * Real.1) / ((Real.1 - x) * (Real.1 - x)) =
            Real.1 / ((Real.1 - x) * (Real.1 - x))
        limit(partial(x.pow)) * limit(partial(x.pow)) =
            Real.1 / ((Real.1 - x) * (Real.1 - x))
        converges_to(partial(seq_mul(geom_square_coeff_seq, x.pow)),
            Real.1 / ((Real.1 - x) * (Real.1 - x)))
    }
}

// ---------------------------------------------------------------------------
// The Fibonacci generating function: Σ fib(n) x^n = x/(1-x-x^2).
// ---------------------------------------------------------------------------

/// The coefficient sequence of the polynomial 1 - x - x^2.
define char_coeff(n: Nat) -> Real {
    if n = Nat.0 {
        Real.1
    } else {
        if n = Nat.1 {
            -Real.1
        } else {
            if n = Nat.2 {
                -Real.1
            } else {
                Real.0
            }
        }
    }
}

/// The coefficient of x^n in (1 - x - x^2) Σ fib(n) x^n.
define char_seq(n: Nat) -> Real {
    cauchy_product(fib_seq, char_coeff, n)
}

/// The second Fibonacci number is one.
theorem fib_two {
    fib(Nat.2) = Nat.1
} by {
    fib_suc_suc(Nat.0)
    fib(Nat.0.suc.suc) = fib(Nat.0.suc) + fib(Nat.0)
    fib_zero
    fib_one
    fib(Nat.2) = Nat.1 + Nat.0
    Nat.1 + Nat.0 = Nat.1
    fib(Nat.2) = Nat.1
}

/// The constant term of (1 - x - x^2) Σ fib(n) x^n is zero.
theorem char_seq_zero {
    char_seq(Nat.0) = Real.0
} by {
    char_seq(Nat.0) = cauchy_product(fib_seq, char_coeff, Nat.0)
    convolve_coeff_sum(fib_seq, char_coeff, Nat.0)
    cauchy_product(fib_seq, char_coeff, Nat.0) =
        partial(cauchy_coefficient(fib_seq, char_coeff, Nat.0), Nat.1)
    partial_one(cauchy_coefficient(fib_seq, char_coeff, Nat.0))
    partial(cauchy_coefficient(fib_seq, char_coeff, Nat.0), Nat.1) =
        cauchy_coefficient(fib_seq, char_coeff, Nat.0, Nat.0)
    cauchy_coefficient(fib_seq, char_coeff, Nat.0, Nat.0) =
        fib_seq(Nat.0) * char_coeff(Nat.0 - Nat.0)
    Nat.0 - Nat.0 = Nat.0
    cauchy_coefficient(fib_seq, char_coeff, Nat.0, Nat.0) =
        fib_seq(Nat.0) * char_coeff(Nat.0)
    cauchy_product(fib_seq, char_coeff, Nat.0) = fib_seq(Nat.0) * char_coeff(Nat.0)
    fib_seq(Nat.0) = from_nat[Real](fib(Nat.0))
    fib_zero
    fib(Nat.0) = Nat.0
    from_nat[Real](fib(Nat.0)) = from_nat[Real](Nat.0)
    from_nat_zero[Real]
    from_nat[Real](Nat.0) = Real.0
    fib_seq(Nat.0) = Real.0
    char_coeff(Nat.0) = Real.1
    fib_seq(Nat.0) * char_coeff(Nat.0) = Real.0 * Real.1
    Real.0 * Real.1 = Real.0
    char_seq(Nat.0) = Real.0
}

/// The coefficient of x in (1 - x - x^2) Σ fib(n) x^n is one.
theorem char_seq_one {
    char_seq(Nat.1) = Real.1
} by {
    char_seq(Nat.1) = cauchy_product(fib_seq, char_coeff, Nat.1)
    convolve_coeff_sum(fib_seq, char_coeff, Nat.1)
    cauchy_product(fib_seq, char_coeff, Nat.1) =
        partial(cauchy_coefficient(fib_seq, char_coeff, Nat.1), Nat.2)
    partial_split_last(cauchy_coefficient(fib_seq, char_coeff, Nat.1), Nat.1)
    partial(cauchy_coefficient(fib_seq, char_coeff, Nat.1), Nat.2) =
        partial(cauchy_coefficient(fib_seq, char_coeff, Nat.1), Nat.1) +
        cauchy_coefficient(fib_seq, char_coeff, Nat.1, Nat.1)
    partial_one(cauchy_coefficient(fib_seq, char_coeff, Nat.1))
    partial(cauchy_coefficient(fib_seq, char_coeff, Nat.1), Nat.1) =
        cauchy_coefficient(fib_seq, char_coeff, Nat.1, Nat.0)
    cauchy_coefficient(fib_seq, char_coeff, Nat.1, Nat.0) =
        fib_seq(Nat.0) * char_coeff(Nat.1 - Nat.0)
    fib_seq(Nat.0) = Real.0
    char_coeff(Nat.1 - Nat.0) = char_coeff(Nat.1)
    char_coeff(Nat.1) = -Real.1
    cauchy_coefficient(fib_seq, char_coeff, Nat.1, Nat.0) = Real.0
    partial(cauchy_coefficient(fib_seq, char_coeff, Nat.1), Nat.1) = Real.0
    cauchy_coefficient(fib_seq, char_coeff, Nat.1, Nat.1) =
        fib_seq(Nat.1) * char_coeff(Nat.1 - Nat.1)
    fib_seq(Nat.1) = from_nat[Real](fib(Nat.1))
    fib_one
    fib(Nat.1) = Nat.1
    from_nat[Real](fib(Nat.1)) = from_nat[Real](Nat.1)
    from_nat_one[Real]
    from_nat[Real](Nat.1) = Real.1
    fib_seq(Nat.1) = Real.1
    char_coeff(Nat.1 - Nat.1) = char_coeff(Nat.0)
    char_coeff(Nat.0) = Real.1
    cauchy_coefficient(fib_seq, char_coeff, Nat.1, Nat.1) = Real.1 * Real.1
    Real.1 * Real.1 = Real.1
    cauchy_coefficient(fib_seq, char_coeff, Nat.1, Nat.1) = Real.1
    partial(cauchy_coefficient(fib_seq, char_coeff, Nat.1), Nat.2) = Real.0 + Real.1
    Real.0 + Real.1 = Real.1
    partial(cauchy_coefficient(fib_seq, char_coeff, Nat.1), Nat.2) = Real.1
    cauchy_product(fib_seq, char_coeff, Nat.1) = Real.1
    char_seq(Nat.1) = Real.1
}

/// The coefficient of x^2 in (1 - x - x^2) Σ fib(n) x^n is zero.
theorem char_seq_two {
    char_seq(Nat.2) = Real.0
} by {
    char_seq(Nat.2) = cauchy_product(fib_seq, char_coeff, Nat.2)
    convolve_coeff_sum(fib_seq, char_coeff, Nat.2)
    cauchy_product(fib_seq, char_coeff, Nat.2) =
        partial(cauchy_coefficient(fib_seq, char_coeff, Nat.2), Nat.3)
    partial_split_last(cauchy_coefficient(fib_seq, char_coeff, Nat.2), Nat.2)
    partial(cauchy_coefficient(fib_seq, char_coeff, Nat.2), Nat.3) =
        partial(cauchy_coefficient(fib_seq, char_coeff, Nat.2), Nat.2) +
        cauchy_coefficient(fib_seq, char_coeff, Nat.2, Nat.2)
    partial_split_last(cauchy_coefficient(fib_seq, char_coeff, Nat.2), Nat.1)
    partial(cauchy_coefficient(fib_seq, char_coeff, Nat.2), Nat.2) =
        partial(cauchy_coefficient(fib_seq, char_coeff, Nat.2), Nat.1) +
        cauchy_coefficient(fib_seq, char_coeff, Nat.2, Nat.1)
    partial_one(cauchy_coefficient(fib_seq, char_coeff, Nat.2))
    partial(cauchy_coefficient(fib_seq, char_coeff, Nat.2), Nat.1) =
        cauchy_coefficient(fib_seq, char_coeff, Nat.2, Nat.0)
    cauchy_coefficient(fib_seq, char_coeff, Nat.2, Nat.0) =
        fib_seq(Nat.0) * char_coeff(Nat.2 - Nat.0)
    fib_seq(Nat.0) = Real.0
    cauchy_coefficient(fib_seq, char_coeff, Nat.2, Nat.0) = Real.0
    partial(cauchy_coefficient(fib_seq, char_coeff, Nat.2), Nat.1) = Real.0
    cauchy_coefficient(fib_seq, char_coeff, Nat.2, Nat.1) =
        fib_seq(Nat.1) * char_coeff(Nat.2 - Nat.1)
    fib_seq(Nat.1) = from_nat[Real](fib(Nat.1))
    fib_one
    fib(Nat.1) = Nat.1
    from_nat[Real](fib(Nat.1)) = from_nat[Real](Nat.1)
    from_nat_one[Real]
    from_nat[Real](Nat.1) = Real.1
    fib_seq(Nat.1) = Real.1
    char_coeff(Nat.2 - Nat.1) = char_coeff(Nat.1)
    char_coeff(Nat.1) = -Real.1
    cauchy_coefficient(fib_seq, char_coeff, Nat.2, Nat.1) = Real.1 * -Real.1
    Real.1 * -Real.1 = -Real.1
    cauchy_coefficient(fib_seq, char_coeff, Nat.2, Nat.1) = -Real.1
    partial(cauchy_coefficient(fib_seq, char_coeff, Nat.2), Nat.2) = Real.0 + -Real.1
    Real.0 + -Real.1 = -Real.1
    partial(cauchy_coefficient(fib_seq, char_coeff, Nat.2), Nat.2) = -Real.1
    cauchy_coefficient(fib_seq, char_coeff, Nat.2, Nat.2) =
        fib_seq(Nat.2) * char_coeff(Nat.2 - Nat.2)
    fib_seq(Nat.2) = from_nat[Real](fib(Nat.2))
    fib_two
    fib(Nat.2) = Nat.1
    from_nat[Real](fib(Nat.2)) = from_nat[Real](Nat.1)
    from_nat_one[Real]
    from_nat[Real](Nat.1) = Real.1
    fib_seq(Nat.2) = Real.1
    char_coeff(Nat.2 - Nat.2) = char_coeff(Nat.0)
    char_coeff(Nat.0) = Real.1
    cauchy_coefficient(fib_seq, char_coeff, Nat.2, Nat.2) = Real.1 * Real.1
    Real.1 * Real.1 = Real.1
    cauchy_coefficient(fib_seq, char_coeff, Nat.2, Nat.2) = Real.1
    partial(cauchy_coefficient(fib_seq, char_coeff, Nat.2), Nat.3) = -Real.1 + Real.1
    -Real.1 + Real.1 = Real.0
    partial(cauchy_coefficient(fib_seq, char_coeff, Nat.2), Nat.3) = Real.0
    cauchy_product(fib_seq, char_coeff, Nat.2) = Real.0
    char_seq(Nat.2) = Real.0
}

/// The coefficients of 1 - x - x^2 vanish from index 3 onwards.
theorem char_coeff_high(m: Nat) {
    not m < Nat.3 implies char_coeff(m) = Real.0
} by {
    if not m < Nat.3 {
        if m = Nat.0 {
            Nat.0 < Nat.3
            m < Nat.3
            false
        }
        if m = Nat.1 {
            Nat.1 < Nat.2
            Nat.2 < Nat.3
            Nat.1 < Nat.2 and Nat.2 < Nat.3
            Nat.1 < Nat.3
            m < Nat.3
            false
        }
        if m = Nat.2 {
            Nat.2 < Nat.3
            m < Nat.3
            false
        }
        if not m = Nat.0 and not m = Nat.1 and not m = Nat.2 {
            char_coeff(m) = Real.0
        }
        char_coeff(m) = Real.0
    }
}

/// The partial sums of the zero sequence vanish.
theorem zero_seq_partial(n: Nat) {
    partial(constant[Nat, Real](Real.0), n) = Real.0
} by {
    define p(k: Nat) -> Bool {
        partial(constant[Nat, Real](Real.0), k) = Real.0
    }
    partial_zero(constant[Nat, Real](Real.0))
    partial(constant[Nat, Real](Real.0), Nat.0) = Real.0
    p(Nat.0)
    forall(k: Nat) {
        if p(k) {
            partial_split_last(constant[Nat, Real](Real.0), k)
            partial(constant[Nat, Real](Real.0), k.suc) =
                partial(constant[Nat, Real](Real.0), k) + constant[Nat, Real](Real.0, k)
            partial(constant[Nat, Real](Real.0), k) = Real.0
            constant[Nat, Real](Real.0, k) = Real.0
            partial(constant[Nat, Real](Real.0), k.suc) = Real.0 + Real.0
            Real.0 + Real.0 = Real.0
            partial(constant[Nat, Real](Real.0), k.suc) = Real.0
            p(k.suc)
        }
    }
    p(Nat.0) and forall(k: Nat) { p(k) implies p(k.suc) }
    alt_induction(p)
    forall(k: Nat) { p(k) }
    p(n)
}

/// For n >= 3, the coefficient of x^n in (1 - x - x^2) Σ fib(n) x^n is zero:
/// the recurrence fib(n+3) = fib(n+2) + fib(n+1) makes the terms cancel.
theorem char_seq_high(n: Nat) {
    char_seq(n.suc.suc.suc) = Real.0
} by {
    char_seq(n.suc.suc.suc) = cauchy_product(fib_seq, char_coeff, n.suc.suc.suc)
    convolve_coeff_sum(fib_seq, char_coeff, n.suc.suc.suc)
    cauchy_product(fib_seq, char_coeff, n.suc.suc.suc) =
        partial(cauchy_coefficient(fib_seq, char_coeff, n.suc.suc.suc), n.suc.suc.suc.suc)
    // Peel off the last three terms: indices n+1, n+2, n+3.
    partial_split_last(cauchy_coefficient(fib_seq, char_coeff, n.suc.suc.suc), n.suc.suc.suc)
    partial(cauchy_coefficient(fib_seq, char_coeff, n.suc.suc.suc), n.suc.suc.suc.suc) =
        partial(cauchy_coefficient(fib_seq, char_coeff, n.suc.suc.suc), n.suc.suc.suc) +
        cauchy_coefficient(fib_seq, char_coeff, n.suc.suc.suc, n.suc.suc.suc)
    partial_split_last(cauchy_coefficient(fib_seq, char_coeff, n.suc.suc.suc), n.suc.suc)
    partial(cauchy_coefficient(fib_seq, char_coeff, n.suc.suc.suc), n.suc.suc.suc) =
        partial(cauchy_coefficient(fib_seq, char_coeff, n.suc.suc.suc), n.suc.suc) +
        cauchy_coefficient(fib_seq, char_coeff, n.suc.suc.suc, n.suc.suc)
    partial_split_last(cauchy_coefficient(fib_seq, char_coeff, n.suc.suc.suc), n.suc)
    partial(cauchy_coefficient(fib_seq, char_coeff, n.suc.suc.suc), n.suc.suc) =
        partial(cauchy_coefficient(fib_seq, char_coeff, n.suc.suc.suc), n.suc) +
        cauchy_coefficient(fib_seq, char_coeff, n.suc.suc.suc, n.suc)
    // The first n+1 terms (indices 0..n) all have char_coeff(n+3-k) = 0.
    forall(k: Nat) {
        if k < n.suc {
            k <= n
            cauchy_coefficient(fib_seq, char_coeff, n.suc.suc.suc, k) =
                fib_seq(k) * char_coeff(n.suc.suc.suc - k)
            add_sub(n, k)
            n - k + k = n
            (n - k + k).suc = n.suc
            (n - k + k).suc = (n - k).suc + k
            (n - k).suc + k = n.suc
            add_imp_sub_left((n - k).suc, k, n.suc)
            n.suc - k = (n - k).suc
            k <= n
            n <= n.suc
            k <= n.suc
            add_sub(n.suc, k)
            n.suc - k + k = n.suc
            (n.suc - k + k).suc = n.suc.suc
            (n.suc - k + k).suc = (n.suc - k).suc + k
            (n.suc - k).suc + k = n.suc.suc
            add_imp_sub_left((n.suc - k).suc, k, n.suc.suc)
            n.suc.suc - k = (n.suc - k).suc
            n.suc <= n.suc.suc
            k <= n.suc.suc
            add_sub(n.suc.suc, k)
            n.suc.suc - k + k = n.suc.suc
            (n.suc.suc - k + k).suc = n.suc.suc.suc
            (n.suc.suc - k + k).suc = (n.suc.suc - k).suc + k
            (n.suc.suc - k).suc + k = n.suc.suc.suc
            add_imp_sub_left((n.suc.suc - k).suc, k, n.suc.suc.suc)
            n.suc.suc.suc - k = (n.suc.suc - k).suc
            (n - k).suc.suc.suc = n.suc.suc.suc - k
            Nat.0 <= n - k
            lte_suc_suc(Nat.0, n - k)
            Nat.0.suc <= (n - k).suc
            lte_suc_suc(Nat.0.suc, (n - k).suc)
            Nat.0.suc.suc <= (n - k).suc.suc
            lte_suc_suc(Nat.0.suc.suc, (n - k).suc.suc)
            Nat.0.suc.suc.suc <= (n - k).suc.suc.suc
            Nat.3 <= (n - k).suc.suc.suc
            not (n - k).suc.suc.suc < Nat.3
            not n.suc.suc.suc - k < Nat.3
            char_coeff_high(n.suc.suc.suc - k)
            char_coeff(n.suc.suc.suc - k) = Real.0
            fib_seq(k) * char_coeff(n.suc.suc.suc - k) = fib_seq(k) * Real.0
            fib_seq(k) * Real.0 = Real.0
            cauchy_coefficient(fib_seq, char_coeff, n.suc.suc.suc, k) = Real.0
        }
    }
    partial_pointwise_eq[Real](
        cauchy_coefficient(fib_seq, char_coeff, n.suc.suc.suc),
        constant[Nat, Real](Real.0), n.suc)
    partial(cauchy_coefficient(fib_seq, char_coeff, n.suc.suc.suc), n.suc) =
        partial(constant[Nat, Real](Real.0), n.suc)
    zero_seq_partial(n.suc)
    partial(constant[Nat, Real](Real.0), n.suc) = Real.0
    partial(cauchy_coefficient(fib_seq, char_coeff, n.suc.suc.suc), n.suc) = Real.0
    // The last three terms.
    cauchy_coefficient(fib_seq, char_coeff, n.suc.suc.suc, n.suc) =
        fib_seq(n.suc) * char_coeff(n.suc.suc.suc - n.suc)
    n.suc.suc.suc = n.suc + Nat.2
    add_imp_sub_left(n.suc, Nat.2, n.suc.suc.suc)
    n.suc.suc.suc - n.suc = Nat.2
    char_coeff(n.suc.suc.suc - n.suc) = char_coeff(Nat.2)
    char_coeff(Nat.2) = -Real.1
    cauchy_coefficient(fib_seq, char_coeff, n.suc.suc.suc, n.suc) =
        fib_seq(n.suc) * -Real.1
    fib_seq(n.suc) * -Real.1 = -(fib_seq(n.suc) * Real.1)
    fib_seq(n.suc) * Real.1 = fib_seq(n.suc)
    -(fib_seq(n.suc) * Real.1) = -fib_seq(n.suc)
    cauchy_coefficient(fib_seq, char_coeff, n.suc.suc.suc, n.suc) = -fib_seq(n.suc)
    cauchy_coefficient(fib_seq, char_coeff, n.suc.suc.suc, n.suc.suc) =
        fib_seq(n.suc.suc) * char_coeff(n.suc.suc.suc - n.suc.suc)
    n.suc.suc.suc = n.suc.suc + Nat.1
    add_imp_sub_left(n.suc.suc, Nat.1, n.suc.suc.suc)
    n.suc.suc.suc - n.suc.suc = Nat.1
    char_coeff(n.suc.suc.suc - n.suc.suc) = char_coeff(Nat.1)
    char_coeff(Nat.1) = -Real.1
    cauchy_coefficient(fib_seq, char_coeff, n.suc.suc.suc, n.suc.suc) =
        fib_seq(n.suc.suc) * -Real.1
    fib_seq(n.suc.suc) * -Real.1 = -(fib_seq(n.suc.suc) * Real.1)
    fib_seq(n.suc.suc) * Real.1 = fib_seq(n.suc.suc)
    -(fib_seq(n.suc.suc) * Real.1) = -fib_seq(n.suc.suc)
    cauchy_coefficient(fib_seq, char_coeff, n.suc.suc.suc, n.suc.suc) = -fib_seq(n.suc.suc)
    cauchy_coefficient(fib_seq, char_coeff, n.suc.suc.suc, n.suc.suc.suc) =
        fib_seq(n.suc.suc.suc) * char_coeff(n.suc.suc.suc - n.suc.suc.suc)
    sub_self(n.suc.suc.suc)
    n.suc.suc.suc - n.suc.suc.suc = Nat.0
    char_coeff(n.suc.suc.suc - n.suc.suc.suc) = char_coeff(Nat.0)
    char_coeff(Nat.0) = Real.1
    cauchy_coefficient(fib_seq, char_coeff, n.suc.suc.suc, n.suc.suc.suc) =
        fib_seq(n.suc.suc.suc) * Real.1
    fib_seq(n.suc.suc.suc) * Real.1 = fib_seq(n.suc.suc.suc)
    cauchy_coefficient(fib_seq, char_coeff, n.suc.suc.suc, n.suc.suc.suc) =
        fib_seq(n.suc.suc.suc)
    // Sum the last three terms using the recurrence fib(n+3) = fib(n+2) + fib(n+1).
    fib_seq(n.suc.suc.suc) = from_nat[Real](fib(n.suc.suc.suc))
    fib_suc_suc(n.suc)
    fib(n.suc.suc.suc) = fib(n.suc.suc) + fib(n.suc)
    from_nat_add[Real](fib(n.suc.suc), fib(n.suc))
    from_nat[Real](fib(n.suc.suc) + fib(n.suc)) =
        from_nat[Real](fib(n.suc.suc)) + from_nat[Real](fib(n.suc))
    fib_seq(n.suc.suc) = from_nat[Real](fib(n.suc.suc))
    fib_seq(n.suc) = from_nat[Real](fib(n.suc))
    fib_seq(n.suc.suc.suc) = fib_seq(n.suc.suc) + fib_seq(n.suc)
    fib_seq(n.suc.suc) + fib_seq(n.suc) = fib_seq(n.suc) + fib_seq(n.suc.suc)
    fib_seq(n.suc.suc.suc) = fib_seq(n.suc) + fib_seq(n.suc.suc)
    partial(cauchy_coefficient(fib_seq, char_coeff, n.suc.suc.suc), n.suc.suc) =
        Real.0 + -fib_seq(n.suc)
    Real.0 + -fib_seq(n.suc) = -fib_seq(n.suc)
    partial(cauchy_coefficient(fib_seq, char_coeff, n.suc.suc.suc), n.suc.suc) =
        -fib_seq(n.suc)
    partial(cauchy_coefficient(fib_seq, char_coeff, n.suc.suc.suc), n.suc.suc.suc) =
        -fib_seq(n.suc) + -fib_seq(n.suc.suc)
    partial(cauchy_coefficient(fib_seq, char_coeff, n.suc.suc.suc), n.suc.suc.suc.suc) =
        (-fib_seq(n.suc) + -fib_seq(n.suc.suc)) + fib_seq(n.suc.suc.suc)
    fib_seq(n.suc) + fib_seq(n.suc.suc) = fib_seq(n.suc.suc.suc)
    neg_pair_cancel_ring(fib_seq(n.suc), fib_seq(n.suc.suc))
    (-fib_seq(n.suc) + -fib_seq(n.suc.suc)) + (fib_seq(n.suc) + fib_seq(n.suc.suc)) = Real.0
    (-fib_seq(n.suc) + -fib_seq(n.suc.suc)) + fib_seq(n.suc.suc.suc) = Real.0
    partial(cauchy_coefficient(fib_seq, char_coeff, n.suc.suc.suc), n.suc.suc.suc.suc) = Real.0
    cauchy_product(fib_seq, char_coeff, n.suc.suc.suc) = Real.0
    char_seq(n.suc.suc.suc) = Real.0
}

// ---------------------------------------------------------------------------
// The analytic form of the Fibonacci generating function.
//
// The coefficient-level statements above (char_seq_zero, char_seq_one,
// char_seq_two, char_seq_high) prove the formal identity
//
//     (1 - x - x^2) * Σ fib(n) x^n = x
//
// coefficient by coefficient: the sequence char_seq(n), the coefficient of
// x^n on the left, is 0, 1, 0, 0, ...  The analytic version of the same
// identity, with convergence, is:
//
//   // theorem fib_gf_analytic(x: Real) {
//   //     x.abs < Real.one_half implies
//   //     converges_to(partial(seq_mul(fib_seq, x.pow)),
//   //         x / (Real.1 - x - x * x))
//   // }
//
// Proving it needs three further ingredients that are not yet in this file:
// an exponential bound fib(n) <= 2^n (to get absolute convergence of
// Σ fib(n) x^n for |x| < 1/2), a convergence proof for the finitely
// supported series Σ char_coeff(n) x^n, and the limit identity
// limit(partial(seq_mul(char_seq, x.pow))) = x.  Each is a routine but
// lengthy analytic argument; the formal (coefficient-level) identity above
// is complete.
