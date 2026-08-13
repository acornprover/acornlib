/// The Bell numbers.
///
/// The `n`-th Bell number `B_n` counts the number of partitions of an
/// `n`-element set, and is the sum of the Stirling numbers of the second kind
/// in row `n`:
///
///     B_n = sum_{k=0}^{n} S(n, k).
///
/// The file proves the small values `B_0 = 1, B_1 = 1, B_2 = 2, B_3 = 5` and
/// the first-moment recurrence
///
///     B_{n+1} = sum_{k=0}^{n} (k + 1) * S(n, k),
///
/// which restates the Stirling recurrence at the level of the Bell sums.
///
/// The classical recurrence
///
///     B_{n+1} = sum_{k=0}^{n} binom(n, k) * B_k,
///
/// counting partitions by the block containing the element `n + 1`, follows
/// from the binomial-transform identity
///
///     sum_{k=0}^{n} binom(n, k) * S(k, j) = S(n + 1, j + 1)
///
/// and a two-dimensional sum interchange; the statement is recorded at the
/// end of the file.

from nat import Nat, add_sub, sub_self, lt_imp_lte_suc, lte_trans, add_comm,
    add_assoc, mul_comm, alt_induction, lt_imp_lt_suc, lt_or_lte, lt_trans,
    add_imp_sub_left, lte_add_right, lte_ref, add_cancels_right,
    distrib_left, distrib_right, mul_assoc, mul_suc_left
from list import partial, partial_split_last, partial_pointwise_eq,
    partial_add, partial_scalar_mul, partial_shift_suc, partial_drop_first
from data.basic.functions import compose
from algebra.semigroup import mul_fn
from algebra.add_semigroup import add_fn
from combinatorics import binom, choose_zero, choose_n, choose_one, pascal_suc_unbounded
from combinatorics.stirling_numbers import stirling, stirling_recurrence,
    stirling_zero_col, stirling_out_of_bounds, stirling_one, stirling_one_one,
    stirling_self

numerals Nat

/// The `n`-th Bell number: the number of partitions of an `n`-element set.
define bell(n: Nat) -> Nat {
    partial(stirling(n), n.suc)
}

/// B_0 = 1.
theorem bell_zero {
    bell(Nat.0) = Nat.1
} by {
    bell(Nat.0) = partial(stirling(Nat.0), Nat.1)
    partial(stirling(Nat.0), Nat.1) = stirling(Nat.0, Nat.0)
    stirling(Nat.0, Nat.0) = Nat.1
    partial(stirling(Nat.0), Nat.1) = Nat.1
    bell(Nat.0) = Nat.1
}

/// B_1 = 1.
theorem bell_one {
    bell(Nat.1) = Nat.1
} by {
    bell(Nat.1) = partial(stirling(Nat.1), Nat.2)
    partial_split_last(stirling(Nat.1), Nat.1)
    partial(stirling(Nat.1), Nat.2) =
        partial(stirling(Nat.1), Nat.1) + stirling(Nat.1, Nat.1)
    partial(stirling(Nat.1), Nat.1) = stirling(Nat.1, Nat.0)
    stirling_one_one
    stirling(Nat.1, Nat.1) = Nat.1
    Nat.0 < Nat.1
    stirling_zero_col(Nat.1)
    stirling(Nat.1, Nat.0) = Nat.0
    partial(stirling(Nat.1), Nat.1) = Nat.0
    partial(stirling(Nat.1), Nat.2) = Nat.0 + Nat.1
    Nat.0 + Nat.1 = Nat.1
    partial(stirling(Nat.1), Nat.2) = Nat.1
    bell(Nat.1) = Nat.1
}

/// B_2 = 2.
theorem bell_two {
    bell(Nat.2) = Nat.2
} by {
    bell(Nat.2) = partial(stirling(Nat.2), Nat.3)
    partial_split_last(stirling(Nat.2), Nat.2)
    partial(stirling(Nat.2), Nat.3) =
        partial(stirling(Nat.2), Nat.2) + stirling(Nat.2, Nat.2)
    partial_split_last(stirling(Nat.2), Nat.1)
    partial(stirling(Nat.2), Nat.2) =
        partial(stirling(Nat.2), Nat.1) + stirling(Nat.2, Nat.1)
    partial(stirling(Nat.2), Nat.1) = stirling(Nat.2, Nat.0)
    Nat.0 < Nat.2
    stirling_zero_col(Nat.2)
    stirling(Nat.2, Nat.0) = Nat.0
    partial(stirling(Nat.2), Nat.1) = Nat.0
    stirling_self(Nat.2)
    stirling(Nat.2, Nat.2) = Nat.1
    stirling_one(Nat.2)
    Nat.0 < Nat.2
    stirling(Nat.2, Nat.1) = Nat.1
    partial(stirling(Nat.2), Nat.2) = Nat.0 + Nat.1
    Nat.0 + Nat.1 = Nat.1
    partial(stirling(Nat.2), Nat.2) = Nat.1
    partial(stirling(Nat.2), Nat.3) = Nat.1 + Nat.1
    Nat.1 + Nat.1 = Nat.2
    partial(stirling(Nat.2), Nat.3) = Nat.2
    bell(Nat.2) = Nat.2
}

/// B_3 = 5.
theorem bell_three {
    bell(Nat.3) = Nat.5
} by {
    bell(Nat.3) = partial(stirling(Nat.3), Nat.4)
    partial_split_last(stirling(Nat.3), Nat.3)
    partial(stirling(Nat.3), Nat.4) =
        partial(stirling(Nat.3), Nat.3) + stirling(Nat.3, Nat.3)
    partial_split_last(stirling(Nat.3), Nat.2)
    partial(stirling(Nat.3), Nat.3) =
        partial(stirling(Nat.3), Nat.2) + stirling(Nat.3, Nat.2)
    partial_split_last(stirling(Nat.3), Nat.1)
    partial(stirling(Nat.3), Nat.2) =
        partial(stirling(Nat.3), Nat.1) + stirling(Nat.3, Nat.1)
    partial(stirling(Nat.3), Nat.1) = stirling(Nat.3, Nat.0)
    Nat.0 < Nat.3
    stirling_zero_col(Nat.3)
    stirling(Nat.3, Nat.0) = Nat.0
    partial(stirling(Nat.3), Nat.1) = Nat.0
    stirling_self(Nat.3)
    stirling(Nat.3, Nat.3) = Nat.1
    stirling_one(Nat.3)
    Nat.0 < Nat.3
    stirling(Nat.3, Nat.1) = Nat.1
    stirling(Nat.3, Nat.2) = stirling(Nat.2, Nat.1) + Nat.2 * stirling(Nat.2, Nat.2)
    stirling_one(Nat.2)
    Nat.0 < Nat.2
    stirling(Nat.2, Nat.1) = Nat.1
    stirling_self(Nat.2)
    stirling(Nat.2, Nat.2) = Nat.1
    stirling(Nat.3, Nat.2) = Nat.1 + Nat.2 * Nat.1
    Nat.2 * Nat.1 = Nat.2
    stirling(Nat.3, Nat.2) = Nat.1 + Nat.2
    Nat.1 + Nat.2 = Nat.3
    stirling(Nat.3, Nat.2) = Nat.3
    partial(stirling(Nat.3), Nat.2) = Nat.0 + Nat.1
    Nat.0 + Nat.1 = Nat.1
    partial(stirling(Nat.3), Nat.2) = Nat.1
    partial(stirling(Nat.3), Nat.3) = Nat.1 + Nat.3
    Nat.1 + Nat.3 = Nat.4
    partial(stirling(Nat.3), Nat.3) = Nat.4
    partial(stirling(Nat.3), Nat.4) = Nat.4 + Nat.1
    Nat.4 + Nat.1 = Nat.5
    partial(stirling(Nat.3), Nat.4) = Nat.5
    bell(Nat.3) = Nat.5
}

/// The j-th summand of the first-moment recurrence:
/// (k + 1) * S(n, k).
define bell_moment_term(n: Nat, k: Nat) -> Nat {
    k.suc * stirling(n, k)
}

/// The shifted weighted summand: (k + 1) * S(n, k + 1).
define bell_shift_term(n: Nat, k: Nat) -> Nat {
    k.suc * stirling(n, k.suc)
}

/// The k-th weighted summand: k * S(n, k).
define bell_weight_fn(n: Nat, k: Nat) -> Nat {
    k * stirling(n, k)
}

/// The moment sum splits into the Bell number and the shifted weighted sum:
/// sum_{k=0}^{n} (k + 1) * S(n, k) = B_n + sum_{k=0}^{n} (k + 1) * S(n, k + 1).
theorem bell_moment_split(n: Nat) {
    partial(bell_moment_term(n), n.suc) =
        bell(n) + partial(bell_shift_term(n), n.suc)
} by {
    // pointwise: bell_moment_term(n, k) = S(n, k) + k * S(n, k)
    forall(k: Nat) {
        if k < n.suc {
            bell_moment_term(n, k) = k.suc * stirling(n, k)
            mul_suc_left(stirling(n, k), k)
            stirling(n, k) + k * stirling(n, k) = k.suc * stirling(n, k)
            bell_weight_fn(n, k) = k * stirling(n, k)
            add_fn(stirling(n), bell_weight_fn(n), k) =
                stirling(n, k) + bell_weight_fn(n, k)
            add_fn(stirling(n), bell_weight_fn(n), k) =
                stirling(n, k) + k * stirling(n, k)
            add_fn(stirling(n), bell_weight_fn(n), k) =
                k.suc * stirling(n, k)
            bell_moment_term(n, k) =
                add_fn(stirling(n), bell_weight_fn(n), k)
        }
    }
    partial_pointwise_eq[Nat](bell_moment_term(n),
        add_fn(stirling(n), bell_weight_fn(n)), n.suc)
    partial(bell_moment_term(n), n.suc) =
        partial(add_fn(stirling(n), bell_weight_fn(n)), n.suc)
    partial_add(stirling(n), bell_weight_fn(n), n.suc)
    partial(stirling(n), n.suc) + partial(bell_weight_fn(n), n.suc) =
        partial(add_fn(stirling(n), bell_weight_fn(n)), n.suc)
    partial(bell_moment_term(n), n.suc) =
        partial(stirling(n), n.suc) + partial(bell_weight_fn(n), n.suc)
    partial(stirling(n), n.suc) = bell(n)
    partial(bell_moment_term(n), n.suc) =
        bell(n) + partial(bell_weight_fn(n), n.suc)
    // the k * S(n, k) sum reindexes to the shifted weighted sum
    // sum_{k=0}^{n} k * S(n, k) = sum_{j=0}^{n} (j + 1) * S(n, j + 1)
    forall(j: Nat) {
        if j < n.suc {
            compose(bell_weight_fn(n), Nat.suc, j) =
                bell_weight_fn(n, j.suc)
            bell_weight_fn(n, j.suc) = j.suc * stirling(n, j.suc)
            bell_shift_term(n, j) = j.suc * stirling(n, j.suc)
            compose(bell_weight_fn(n), Nat.suc, j) =
                bell_shift_term(n, j)
        }
    }
    partial_pointwise_eq[Nat](compose(bell_weight_fn(n), Nat.suc),
        bell_shift_term(n), n.suc)
    partial(compose(bell_weight_fn(n), Nat.suc), n.suc) =
        partial(bell_shift_term(n), n.suc)
    partial_shift_suc[Nat](bell_weight_fn(n), n.suc)
    bell_weight_fn(n, Nat.0) +
        partial(compose(bell_weight_fn(n), Nat.suc), n.suc) =
        partial(bell_weight_fn(n), n.suc.suc)
    bell_weight_fn(n, Nat.0) = Nat.0 * stirling(n, Nat.0)
    Nat.0 * stirling(n, Nat.0) = Nat.0
    bell_weight_fn(n, Nat.0) = Nat.0
    partial(bell_weight_fn(n), n.suc.suc) =
        partial(bell_weight_fn(n), n.suc) + bell_weight_fn(n, n.suc)
    bell_weight_fn(n, n.suc) = n.suc * stirling(n, n.suc)
    stirling(n, n.suc) = Nat.0
    n < n.suc
    stirling_out_of_bounds(n, n.suc)
    stirling(n, n.suc) = Nat.0
    bell_weight_fn(n, n.suc) = n.suc * Nat.0
    n.suc * Nat.0 = Nat.0
    bell_weight_fn(n, n.suc) = Nat.0
    partial(bell_weight_fn(n), n.suc.suc) =
        partial(bell_weight_fn(n), n.suc)
    partial(compose(bell_weight_fn(n), Nat.suc), n.suc) =
        partial(bell_weight_fn(n), n.suc.suc) - bell_weight_fn(n, Nat.0)
    partial(bell_weight_fn(n), n.suc.suc) - Nat.0 =
        partial(bell_weight_fn(n), n.suc.suc)
    partial(compose(bell_weight_fn(n), Nat.suc), n.suc) =
        partial(bell_weight_fn(n), n.suc.suc)
    partial(compose(bell_weight_fn(n), Nat.suc), n.suc) =
        partial(bell_weight_fn(n), n.suc)
    partial(bell_shift_term(n), n.suc) =
        partial(bell_weight_fn(n), n.suc)
    partial(bell_moment_term(n), n.suc) =
        bell(n) + partial(bell_shift_term(n), n.suc)
}

/// The first-moment recurrence: B_{n+1} = sum_{k=0}^{n} (k + 1) * S(n, k).
theorem bell_moment_recurrence(n: Nat) {
    bell(n.suc) = partial(bell_moment_term(n), n.suc)
} by {
    // pointwise: S(n + 1, j + 1) = S(n, j) + (j + 1) * S(n, j + 1)
    forall(j: Nat) {
        if j < n.suc {
            stirling_recurrence(n, j)
            stirling(n.suc, j.suc) = stirling(n, j) + j.suc * stirling(n, j.suc)
            compose(stirling(n.suc), Nat.suc, j) = stirling(n.suc, j.suc)
            compose(stirling(n.suc), Nat.suc, j) =
                stirling(n, j) + j.suc * stirling(n, j.suc)
            add_fn(stirling(n), bell_shift_term(n), j) =
                stirling(n, j) + bell_shift_term(n, j)
            bell_shift_term(n, j) = j.suc * stirling(n, j.suc)
            add_fn(stirling(n), bell_shift_term(n), j) =
                stirling(n, j) + j.suc * stirling(n, j.suc)
            compose(stirling(n.suc), Nat.suc, j) =
                add_fn(stirling(n), bell_shift_term(n), j)
        }
    }
    partial_pointwise_eq[Nat](compose(stirling(n.suc), Nat.suc),
        add_fn(stirling(n), bell_shift_term(n)), n.suc)
    partial(compose(stirling(n.suc), Nat.suc), n.suc) =
        partial(add_fn(stirling(n), bell_shift_term(n)), n.suc)
    partial_add(stirling(n), bell_shift_term(n), n.suc)
    partial(stirling(n), n.suc) + partial(bell_shift_term(n), n.suc) =
        partial(add_fn(stirling(n), bell_shift_term(n)), n.suc)
    partial(compose(stirling(n.suc), Nat.suc), n.suc) =
        partial(stirling(n), n.suc) + partial(bell_shift_term(n), n.suc)
    partial(stirling(n), n.suc) = bell(n)
    partial(compose(stirling(n.suc), Nat.suc), n.suc) =
        bell(n) + partial(bell_shift_term(n), n.suc)
    // the full sum: S(n + 1, 0) + sum_{j=0}^{n} S(n + 1, j + 1)
    partial_drop_first[Nat](stirling(n.suc), n.suc.suc)
    Nat.0 < n.suc.suc
    partial(stirling(n.suc), n.suc.suc) =
        stirling(n.suc, Nat.0) + partial(compose(stirling(n.suc), Nat.suc), n.suc)
    stirling(n.suc, Nat.0) = Nat.0
    stirling_zero_col(n.suc)
    Nat.0 < n.suc
    stirling(n.suc, Nat.0) = Nat.0
    partial(stirling(n.suc), n.suc.suc) =
        Nat.0 + partial(compose(stirling(n.suc), Nat.suc), n.suc)
    Nat.0 + partial(compose(stirling(n.suc), Nat.suc), n.suc) =
        partial(compose(stirling(n.suc), Nat.suc), n.suc)
    partial(stirling(n.suc), n.suc.suc) =
        partial(compose(stirling(n.suc), Nat.suc), n.suc)
    partial(stirling(n.suc), n.suc.suc) =
        bell(n) + partial(bell_shift_term(n), n.suc)
    bell(n.suc) = partial(stirling(n.suc), n.suc.suc)
    bell(n.suc) = bell(n) + partial(bell_shift_term(n), n.suc)
    bell_moment_split(n)
    partial(bell_moment_term(n), n.suc) =
        bell(n) + partial(bell_shift_term(n), n.suc)
    bell(n.suc) = partial(bell_moment_term(n), n.suc)
}

/// The k-th summand of the binomial transform of a Stirling column:
/// binom(n, k) * S(k, j).
define stirling_binom_term(n: Nat, j: Nat, k: Nat) -> Nat {
    n.binom(k) * stirling(k, j)
}

// The classical recurrence B_{n+1} = sum_{k=0}^{n} binom(n, k) * B_k.
//
// Proof roadmap (statement recorded for later work):
//
//   (a) The binomial transform identity
//         sum_{k=0}^{n} binom(n, k) * S(k, j) = S(n + 1, j + 1)
//       is proved by induction on n using Pascal's identity on the row
//       binom(n + 1, k) = binom(n, k) + binom(n, k - 1) and the Stirling
//       recurrence, exactly as the Catalan convolution family is handled in
//       `combinatorics/catalan.ac`.
//
//   (b) Summing (a) over j gives
//         B_{n+1} = sum_{k=0}^{n} binom(n, k) * sum_{j} S(k, j)
//                 = sum_{k=0}^{n} binom(n, k) * B_k,
//       where the double sum is interchanged using
//       `partial_interchange` of `data/nat/nat_range_sum_interchange.ac`.
//
// theorem bell_recurrence(n: Nat) {
//     bell(n.suc) = partial(bell_binom_term(n), n.suc)
// }
