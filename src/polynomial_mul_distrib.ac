from nat import Nat
from comm_ring import CommRing
from algebra.add_semigroup import add_fn
from data.basic.functions import function_extensionality
from polynomial import Polynomial, polynomial_mul, polynomial_mul_coeff,
    polynomial_mul_term_coeff, polynomial_mul_coeff_apply, polynomial_ext_pointwise,
    polynomial_add_coeff
from data.nat.nat_range_sum import range_sum, range_sum_add
from polynomial_mul_comm import mul_coeff_eq_range_sum

/// The convolution summand against a sum splits.
///
/// Pointwise distributivity in the coefficient ring, read one exponent at a time.
theorem mul_term_coeff_add[R: CommRing](
    p: Polynomial[R], q: Polynomial[R], r: Polynomial[R], k: Nat
) {
    polynomial_mul_term_coeff(p, q + r, k)
        = add_fn(polynomial_mul_term_coeff(p, q, k), polynomial_mul_term_coeff(p, r, k))
} by {
    forall(i: Nat) {
        (polynomial_mul_term_coeff(p, q + r, k, i) = p.coeff(i) * (q + r).coeff(k - i))
        polynomial_add_coeff(q, r, k - i)
        ((q + r).coeff(k - i) = q.coeff(k - i) + r.coeff(k - i))
        (p.coeff(i) * (q.coeff(k - i) + r.coeff(k - i))
            = p.coeff(i) * q.coeff(k - i) + p.coeff(i) * r.coeff(k - i))
        (polynomial_mul_term_coeff(p, q, k, i) = p.coeff(i) * q.coeff(k - i))
        (polynomial_mul_term_coeff(p, r, k, i) = p.coeff(i) * r.coeff(k - i))
        (add_fn(polynomial_mul_term_coeff(p, q, k), polynomial_mul_term_coeff(p, r, k))(i)
            = polynomial_mul_term_coeff(p, q, k)(i)
                + polynomial_mul_term_coeff(p, r, k)(i))
        (polynomial_mul_term_coeff(p, q + r, k)(i)
            = add_fn(polynomial_mul_term_coeff(p, q, k),
                polynomial_mul_term_coeff(p, r, k))(i))
    }
    function_extensionality[Nat, R](polynomial_mul_term_coeff(p, q + r, k),
        add_fn(polynomial_mul_term_coeff(p, q, k), polynomial_mul_term_coeff(p, r, k)))
    (polynomial_mul_term_coeff(p, q + r, k)
        = add_fn(polynomial_mul_term_coeff(p, q, k), polynomial_mul_term_coeff(p, r, k)))
}

/// The convolution coefficient against a sum splits.
theorem polynomial_mul_coeff_add[R: CommRing](
    p: Polynomial[R], q: Polynomial[R], r: Polynomial[R], k: Nat
) {
    polynomial_mul_coeff(p, q + r, k)
        = polynomial_mul_coeff(p, q, k) + polynomial_mul_coeff(p, r, k)
} by {
    mul_coeff_eq_range_sum(p, q + r, k)
    (polynomial_mul_coeff(p, q + r, k)
        = range_sum(polynomial_mul_term_coeff(p, q + r, k), k.suc))
    mul_term_coeff_add(p, q, r, k)
    (polynomial_mul_term_coeff(p, q + r, k)
        = add_fn(polynomial_mul_term_coeff(p, q, k), polynomial_mul_term_coeff(p, r, k)))
    range_sum_add(polynomial_mul_term_coeff(p, q, k),
        polynomial_mul_term_coeff(p, r, k), k.suc)
    (range_sum(add_fn(polynomial_mul_term_coeff(p, q, k),
        polynomial_mul_term_coeff(p, r, k)), k.suc)
        = range_sum(polynomial_mul_term_coeff(p, q, k), k.suc)
            + range_sum(polynomial_mul_term_coeff(p, r, k), k.suc))
    mul_coeff_eq_range_sum(p, q, k)
    (polynomial_mul_coeff(p, q, k)
        = range_sum(polynomial_mul_term_coeff(p, q, k), k.suc))
    mul_coeff_eq_range_sum(p, r, k)
    (polynomial_mul_coeff(p, r, k)
        = range_sum(polynomial_mul_term_coeff(p, r, k), k.suc))
    (polynomial_mul_coeff(p, q + r, k)
        = polynomial_mul_coeff(p, q, k) + polynomial_mul_coeff(p, r, k))
}

/// Polynomial multiplication distributes over addition on the left.
///
/// The last of the ring axioms, and the cheapest: the convolution against a sum is the pointwise
/// sum of the two convolutions, so the range sum splits with no reindexing at all.
theorem polynomial_mul_add_left[R: CommRing](
    p: Polynomial[R], q: Polynomial[R], r: Polynomial[R]
) {
    polynomial_mul(p, q + r) = polynomial_mul(p, q) + polynomial_mul(p, r)
} by {
    forall(k: Nat) {
        polynomial_mul_coeff_apply(p, q + r, k)
        (polynomial_mul(p, q + r).coeff(k) = polynomial_mul_coeff(p, q + r, k))
        polynomial_mul_coeff_add(p, q, r, k)
        (polynomial_mul_coeff(p, q + r, k)
            = polynomial_mul_coeff(p, q, k) + polynomial_mul_coeff(p, r, k))
        polynomial_mul_coeff_apply(p, q, k)
        (polynomial_mul(p, q).coeff(k) = polynomial_mul_coeff(p, q, k))
        polynomial_mul_coeff_apply(p, r, k)
        (polynomial_mul(p, r).coeff(k) = polynomial_mul_coeff(p, r, k))
        polynomial_add_coeff(polynomial_mul(p, q), polynomial_mul(p, r), k)
        ((polynomial_mul(p, q) + polynomial_mul(p, r)).coeff(k)
            = polynomial_mul(p, q).coeff(k) + polynomial_mul(p, r).coeff(k))
        (polynomial_mul(p, q + r).coeff(k)
            = (polynomial_mul(p, q) + polynomial_mul(p, r)).coeff(k))
    }
    polynomial_ext_pointwise(polynomial_mul(p, q + r),
        polynomial_mul(p, q) + polynomial_mul(p, r))
    polynomial_mul(p, q + r) = polynomial_mul(p, q) + polynomial_mul(p, r)
}
