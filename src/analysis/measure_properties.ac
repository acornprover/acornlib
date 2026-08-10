/// Measure theory properties on top of the outer-measure API: monotonicity,
/// the measure of the empty set, additivity for disjoint measurable sets,
/// the inclusion-exclusion identity for two sets, and the Lebesgue measure
/// of intervals.
///
/// The abstract results are stated for an arbitrary `OuterMeasure[K]` from
/// `analysis.topology.measure`; additivity requires Carathéodory
/// measurability, exactly as in the classical theory (an outer measure is
/// merely subadditive in general).  The Lebesgue versions instantiate the
/// same arguments on `lebesgue_outer_measure` from
/// `analysis.topology.lebesgue`.

from data.basic.set import Set, set_ext, set_eq_contains_at,
    empty_set_contains_eq, union_contains_eq, intersection_contains_eq,
    difference_contains_eq, union_comm, intersection_comm, sets_subset_union,
    intersection_with_superset_is_self, range_indexed_union,
    range_indexed_union_contains_witness, range_indexed_union_zero
from nat import Nat, lt_suc, lt_imp_lt_suc, alt_induction
from real import Real, sub_moves_sides, pos_gt_zero
from order_set import closed_interval_set
from analysis.topology.measure import OuterMeasure, caratheodory_measurable,
    outer_measure_monotone, outer_measure_empty, outer_measure_nonneg,
    measure_seq, range_measure, range_measure_zero, range_measure_suc,
    range_indexed_union_suc
from analysis.topology.lebesgue import lebesgue_outer_measure,
    lebesgue_cost_has_infimum, lebesgue_measurable,
    lebesgue_outer_measure_empty, lebesgue_outer_measure_monotone,
    lebesgue_outer_measure_nonneg, lebesgue_outer_measure_interval_upper,
    lebesgue_outer_measure_point, interval_cost_has_infimum

numerals Real

// Set lemmas local to the measure properties; they are stated in the same
// style as the set facts in `analysis.topology.measure.ac`.

/// The intersection of a union with its left operand is the left operand.
theorem union_intersection_absorb_left[K](a: Set[K], b: Set[K]) {
    (a.union(b)).intersection(a) = a
} by {
    sets_subset_union(a, b)
    a.subset(a.union(b))
    intersection_with_superset_is_self(a, a.union(b))
    a.intersection(a.union(b)) = a
    intersection_comm(a, a.union(b))
    a.intersection(a.union(b)) = a.union(b).intersection(a)
    (a.union(b)).intersection(a) = a
}

/// The difference of a union with its left operand is the right operand
/// minus the left operand.
theorem union_difference_left[K](a: Set[K], b: Set[K]) {
    (a.union(b)).difference(a) = b.difference(a)
} by {
    forall(x: K) {
        difference_contains_eq(a.union(b), a, x)
        union_contains_eq(a, b, x)
        difference_contains_eq(b, a, x)
        if (a.union(b)).difference(a).contains(x) {
            b.difference(a).contains(x)
        }
        if b.difference(a).contains(x) {
            (a.union(b)).difference(a).contains(x)
        }
        (a.union(b)).difference(a).contains(x) = b.difference(a).contains(x)
    }
    set_ext((a.union(b)).difference(a), b.difference(a))
}

/// Disjoint sets have no element in common.
theorem disjoint_imp_not_both[K](a: Set[K], b: Set[K], x: K) {
    a.intersection(b) = Set[K].empty_set implies not (a.contains(x) and b.contains(x))
} by {
    if a.intersection(b) = Set[K].empty_set {
        set_eq_contains_at(a.intersection(b), Set[K].empty_set, x)
        a.intersection(b).contains(x) = Set[K].empty_set.contains(x)
        empty_set_contains_eq[K](x)
        Set[K].empty_set.contains(x) = false
        a.intersection(b).contains(x) = false
        intersection_contains_eq(a, b, x)
        (a.contains(x) and b.contains(x)) = false
        not (a.contains(x) and b.contains(x))
    }
}

/// Membership in a set difference with a disjoint set is membership in the
/// set itself.
theorem difference_of_disjoint_pointwise[K](a: Set[K], b: Set[K], x: K) {
    a.intersection(b) = Set[K].empty_set implies (b.difference(a).contains(x) = b.contains(x))
} by {
    if a.intersection(b) = Set[K].empty_set {
        difference_contains_eq(b, a, x)
        if b.difference(a).contains(x) {
            b.contains(x)
        }
        if b.contains(x) {
            disjoint_imp_not_both(a, b, x)
            not (a.contains(x) and b.contains(x))
            if a.contains(x) {
                a.contains(x) and b.contains(x)
                false
            }
            not a.contains(x)
            b.difference(a).contains(x)
        }
        b.difference(a).contains(x) = b.contains(x)
    }
}

/// A set difference with a disjoint set is the set itself.
theorem difference_of_disjoint_is_self[K](a: Set[K], b: Set[K]) {
    a.intersection(b) = Set[K].empty_set implies b.difference(a) = b
} by {
    if a.intersection(b) = Set[K].empty_set {
        forall(x: K) {
            difference_of_disjoint_pointwise(a, b, x)
            b.difference(a).contains(x) = b.contains(x)
        }
        set_ext(b.difference(a), b)
    }
}

// Measure-theoretic properties of a general outer measure.

/// The measure is monotone: a subset has no larger measure.
theorem measure_monotone[K](m: OuterMeasure[K], a: Set[K], b: Set[K]) {
    a.subset(b) implies m.measure(a) <= m.measure(b)
} by {
    outer_measure_monotone(m, a, b)
}

/// The measure of the empty set is zero.
theorem measure_empty[K](m: OuterMeasure[K]) {
    m.measure(Set[K].empty_set) = Real.0
} by {
    outer_measure_empty(m)
}

/// The measure of every set is nonnegative.
theorem measure_nonneg[K](m: OuterMeasure[K], s: Set[K]) {
    Real.0 <= m.measure(s)
} by {
    outer_measure_nonneg(m, s)
}

/// A Carathéodory-measurable set splits the measure of any union with it:
/// `m(a ∪ b) = m(a) + m(b \ a)`.
theorem caratheodory_measurable_union_splits[K](m: OuterMeasure[K], a: Set[K], b: Set[K]) {
    caratheodory_measurable(m, a) implies
        m.measure(a.union(b)) = m.measure(a) + m.measure(b.difference(a))
} by {
    if caratheodory_measurable(m, a) {
        caratheodory_measurable(m, a) = forall(e: Set[K]) {
            m.measure(e) = m.measure(e.intersection(a)) + m.measure(e.difference(a))
        }
        m.measure(a.union(b)) =
            m.measure((a.union(b)).intersection(a)) + m.measure((a.union(b)).difference(a))
        union_intersection_absorb_left(a, b)
        (a.union(b)).intersection(a) = a
        m.measure((a.union(b)).intersection(a)) = m.measure(a)
        union_difference_left(a, b)
        (a.union(b)).difference(a) = b.difference(a)
        m.measure((a.union(b)).difference(a)) = m.measure(b.difference(a))
        m.measure(a.union(b)) = m.measure(a) + m.measure(b.difference(a))
    }
}

/// The measure of the union of two disjoint sets is the sum of their measures,
/// provided one of them is measurable.
theorem caratheodory_measurable_disjoint_add[K](m: OuterMeasure[K], a: Set[K], b: Set[K]) {
    caratheodory_measurable(m, a) and a.intersection(b) = Set[K].empty_set
        implies m.measure(a.union(b)) = m.measure(a) + m.measure(b)
} by {
    if caratheodory_measurable(m, a) and a.intersection(b) = Set[K].empty_set {
        caratheodory_measurable_union_splits(m, a, b)
        m.measure(a.union(b)) = m.measure(a) + m.measure(b.difference(a))
        difference_of_disjoint_is_self(a, b)
        b.difference(a) = b
        m.measure(b.difference(a)) = m.measure(b)
        m.measure(a.union(b)) = m.measure(a) + m.measure(b)
    }
}

/// The same additivity when the second of the two disjoint sets is measurable.
theorem caratheodory_measurable_disjoint_add_right[K](m: OuterMeasure[K], a: Set[K], b: Set[K]) {
    caratheodory_measurable(m, b) and a.intersection(b) = Set[K].empty_set
        implies m.measure(a.union(b)) = m.measure(a) + m.measure(b)
} by {
    if caratheodory_measurable(m, b) and a.intersection(b) = Set[K].empty_set {
        intersection_comm(a, b)
        a.intersection(b) = b.intersection(a)
        b.intersection(a) = Set[K].empty_set
        caratheodory_measurable_disjoint_add(m, b, a)
        m.measure(b.union(a)) = m.measure(b) + m.measure(a)
        union_comm(b, a)
        b.union(a) = a.union(b)
        m.measure(a.union(b)) = m.measure(b) + m.measure(a)
        m.measure(a.union(b)) = m.measure(a) + m.measure(b)
    }
}

/// The inclusion-exclusion identity for the measure of two measurable sets:
/// `m(a ∪ b) = m(a) + m(b) - m(a ∩ b)`.
theorem caratheodory_measurable_inclusion_exclusion[K](m: OuterMeasure[K], a: Set[K], b: Set[K]) {
    caratheodory_measurable(m, a) and caratheodory_measurable(m, b)
        implies m.measure(a.union(b)) = m.measure(a) + m.measure(b) - m.measure(a.intersection(b))
} by {
    if caratheodory_measurable(m, a) and caratheodory_measurable(m, b) {
        caratheodory_measurable_union_splits(m, a, b)
        m.measure(a.union(b)) = m.measure(a) + m.measure(b.difference(a))
        caratheodory_measurable(m, a) = forall(e: Set[K]) {
            m.measure(e) = m.measure(e.intersection(a)) + m.measure(e.difference(a))
        }
        m.measure(b) = m.measure(b.intersection(a)) + m.measure(b.difference(a))
        intersection_comm(b, a)
        b.intersection(a) = a.intersection(b)
        m.measure(b) = m.measure(a.intersection(b)) + m.measure(b.difference(a))
        m.measure(b.difference(a)) + m.measure(a.intersection(b)) = m.measure(b)
        sub_moves_sides(m.measure(b.difference(a)), m.measure(a.intersection(b)), m.measure(b))
        m.measure(b.difference(a)) = m.measure(b) - m.measure(a.intersection(b))
        m.measure(a.union(b)) = m.measure(a) + (m.measure(b) - m.measure(a.intersection(b)))
        m.measure(a.union(b)) = m.measure(a) + m.measure(b) - m.measure(a.intersection(b))
    }
}

// Properties of the Lebesgue outer measure on the real line.

/// The Lebesgue outer measure is monotone (restated): a subset has no larger
/// measure wherever both cost sets have infima.
theorem lebesgue_measure_monotone(s: Set[Real], t: Set[Real]) {
    s.subset(t) and lebesgue_cost_has_infimum(s) and lebesgue_cost_has_infimum(t)
        implies lebesgue_outer_measure(s) <= lebesgue_outer_measure(t)
} by {
    if s.subset(t) and lebesgue_cost_has_infimum(s) and lebesgue_cost_has_infimum(t) {
        lebesgue_outer_measure_monotone(s, t)
        lebesgue_outer_measure(s) <= lebesgue_outer_measure(t)
    }
}

/// The Lebesgue outer measure of the empty set is zero (restated).
theorem lebesgue_measure_empty {
    lebesgue_outer_measure(Set[Real].empty_set) = Real.0
} by {
    lebesgue_outer_measure_empty
}

/// The Lebesgue outer measure is nonnegative wherever its cost set has an
/// infimum (restated).
theorem lebesgue_measure_nonneg(s: Set[Real]) {
    lebesgue_cost_has_infimum(s) implies Real.0 <= lebesgue_outer_measure(s)
} by {
    if lebesgue_cost_has_infimum(s) {
        lebesgue_outer_measure_nonneg(s)
        Real.0 <= lebesgue_outer_measure(s)
    }
}

/// The Lebesgue outer measure of the union of two disjoint sets is the sum of
/// their measures, provided one of them is Lebesgue measurable.
theorem lebesgue_measure_disjoint_add(a: Set[Real], b: Set[Real]) {
    lebesgue_measurable(a) and a.intersection(b) = Set[Real].empty_set
        implies lebesgue_outer_measure(a.union(b)) =
            lebesgue_outer_measure(a) + lebesgue_outer_measure(b)
} by {
    if lebesgue_measurable(a) and a.intersection(b) = Set[Real].empty_set {
        lebesgue_measurable(a) = forall(e: Set[Real]) {
            lebesgue_outer_measure(e) =
                lebesgue_outer_measure(e.intersection(a)) + lebesgue_outer_measure(e.difference(a))
        }
        lebesgue_outer_measure(a.union(b)) =
            lebesgue_outer_measure((a.union(b)).intersection(a)) +
                lebesgue_outer_measure((a.union(b)).difference(a))
        union_intersection_absorb_left(a, b)
        (a.union(b)).intersection(a) = a
        lebesgue_outer_measure((a.union(b)).intersection(a)) = lebesgue_outer_measure(a)
        union_difference_left(a, b)
        (a.union(b)).difference(a) = b.difference(a)
        lebesgue_outer_measure((a.union(b)).difference(a)) = lebesgue_outer_measure(b.difference(a))
        lebesgue_outer_measure(a.union(b)) =
            lebesgue_outer_measure(a) + lebesgue_outer_measure(b.difference(a))
        difference_of_disjoint_is_self(a, b)
        b.difference(a) = b
        lebesgue_outer_measure(b.difference(a)) = lebesgue_outer_measure(b)
        lebesgue_outer_measure(a.union(b)) =
            lebesgue_outer_measure(a) + lebesgue_outer_measure(b)
    }
}

/// The inclusion-exclusion identity for the Lebesgue outer measure of two
/// measurable sets.
theorem lebesgue_measure_inclusion_exclusion(a: Set[Real], b: Set[Real]) {
    lebesgue_measurable(a) and lebesgue_measurable(b)
        implies lebesgue_outer_measure(a.union(b)) =
            lebesgue_outer_measure(a) + lebesgue_outer_measure(b) -
                lebesgue_outer_measure(a.intersection(b))
} by {
    if lebesgue_measurable(a) and lebesgue_measurable(b) {
        lebesgue_measurable(a) = forall(e: Set[Real]) {
            lebesgue_outer_measure(e) =
                lebesgue_outer_measure(e.intersection(a)) + lebesgue_outer_measure(e.difference(a))
        }
        lebesgue_outer_measure(a.union(b)) =
            lebesgue_outer_measure((a.union(b)).intersection(a)) +
                lebesgue_outer_measure((a.union(b)).difference(a))
        union_intersection_absorb_left(a, b)
        (a.union(b)).intersection(a) = a
        lebesgue_outer_measure((a.union(b)).intersection(a)) = lebesgue_outer_measure(a)
        union_difference_left(a, b)
        (a.union(b)).difference(a) = b.difference(a)
        lebesgue_outer_measure((a.union(b)).difference(a)) = lebesgue_outer_measure(b.difference(a))
        lebesgue_outer_measure(a.union(b)) =
            lebesgue_outer_measure(a) + lebesgue_outer_measure(b.difference(a))
        lebesgue_outer_measure(b) =
            lebesgue_outer_measure(b.intersection(a)) + lebesgue_outer_measure(b.difference(a))
        intersection_comm(b, a)
        b.intersection(a) = a.intersection(b)
        lebesgue_outer_measure(b) =
            lebesgue_outer_measure(a.intersection(b)) + lebesgue_outer_measure(b.difference(a))
        lebesgue_outer_measure(b.difference(a)) + lebesgue_outer_measure(a.intersection(b)) =
            lebesgue_outer_measure(b)
        sub_moves_sides(lebesgue_outer_measure(b.difference(a)),
            lebesgue_outer_measure(a.intersection(b)), lebesgue_outer_measure(b))
        lebesgue_outer_measure(b.difference(a)) =
            lebesgue_outer_measure(b) - lebesgue_outer_measure(a.intersection(b))
        lebesgue_outer_measure(a.union(b)) =
            lebesgue_outer_measure(a) + (lebesgue_outer_measure(b) - lebesgue_outer_measure(a.intersection(b)))
        lebesgue_outer_measure(a.union(b)) =
            lebesgue_outer_measure(a) + lebesgue_outer_measure(b) -
                lebesgue_outer_measure(a.intersection(b))
    }
}

// The measure of intervals.

/// The Lebesgue outer measure of a closed interval is at most its length
/// (the upper half of the defining property, restated).
theorem lebesgue_measure_interval_upper_bound(a: Real, b: Real) {
    a <= b implies lebesgue_outer_measure(closed_interval_set(a, b)) <= b - a
} by {
    lebesgue_outer_measure_interval_upper(a, b)
}

/// The Lebesgue outer measure of the unit interval is bounded by its length:
/// `0 <= μ([0,1]) <= 1`.
theorem lebesgue_measure_unit_interval_bounds {
    Real.0 <= lebesgue_outer_measure(closed_interval_set(Real.0, Real.1))
        and lebesgue_outer_measure(closed_interval_set(Real.0, Real.1)) <= Real.1
} by {
    Real.1.is_positive
    pos_gt_zero(Real.1)
    Real.0 < Real.1
    Real.0 <= Real.1
    interval_cost_has_infimum(Real.0, Real.1)
    lebesgue_cost_has_infimum(closed_interval_set(Real.0, Real.1))
    lebesgue_outer_measure_nonneg(closed_interval_set(Real.0, Real.1))
    Real.0 <= lebesgue_outer_measure(closed_interval_set(Real.0, Real.1))
    lebesgue_outer_measure_interval_upper(Real.0, Real.1)
    lebesgue_outer_measure(closed_interval_set(Real.0, Real.1)) <= Real.1 - Real.0
    lebesgue_outer_measure(closed_interval_set(Real.0, Real.1)) <= Real.1
}

/// The Lebesgue outer measure of a degenerate interval is zero: `μ([0,0]) = 0`.
theorem lebesgue_measure_point_zero {
    lebesgue_outer_measure(closed_interval_set(Real.0, Real.0)) = Real.0
} by {
    lebesgue_outer_measure_point(Real.0)
}

// The defining property of the Lebesgue measure,
//     theorem lebesgue_measure_interval(a: Real, b: Real) {
//         a <= b implies lebesgue_outer_measure(closed_interval_set(a, b)) = b - a
//     }
// is not yet provable: `lebesgue.ac` proves the upper bound
// `lebesgue_outer_measure_interval_upper` and the degenerate case
// `lebesgue_outer_measure_point`, while the reverse inequality (every
// countable interval cover of `[a, b]` has total cost at least `b - a`) is
// documented there as a deferred TODO requiring the Heine–Borel open-cover
// reduction for intervals.  For the unit interval this file verifies the
// available bounds in `lebesgue_measure_unit_interval_bounds` and the
// degenerate case `lebesgue_measure_point_zero`.

// Countable additivity for disjoint measurable families.

// The full classical statement of countable additivity,
//     m.measure(indexed_union(family)) = sum_{n : Nat} m.measure(family(n))
// for a pairwise disjoint family of measurable sets, is not representable in
// the Real-valued setting (the library's reals have no element `infinity` and
// no countable sums; compare the discussion in `analysis.topology.measure.ac`).
// The finite form below — the measure of the union of the first `n` disjoint
// measurable members equals the partial sum of their measures — is the
// representable approximation, and it subsumes the binary additivity proved
// above (`caratheodory_measurable_disjoint_add`).

/// True if the first `n` members of `family` are pairwise disjoint.
define pairwise_disjoint[K](family: Nat -> Set[K], n: Nat) -> Bool {
    forall(i: Nat, j: Nat) {
        i < n and j < n and i < j implies family(i).intersection(family(j)) = Set[K].empty_set
    }
}

/// True if the first `n` members of `family` are Carathéodory measurable.
define measurable_range[K](m: OuterMeasure[K], family: Nat -> Set[K], n: Nat) -> Bool {
    forall(i: Nat) { i < n implies caratheodory_measurable(m, family(i)) }
}

/// No element of the union of the first `k` members lies in the next member
/// when the members below `k.suc` are pairwise disjoint.
theorem range_indexed_union_disjoint_last_contains[K](family: Nat -> Set[K], k: Nat, x: K) {
    pairwise_disjoint(family, k.suc)
        implies not range_indexed_union(k, family).intersection(family(k)).contains(x)
} by {
    if pairwise_disjoint(family, k.suc) {
        pairwise_disjoint(family, k.suc) = forall(i: Nat, j: Nat) {
            i < k.suc and j < k.suc and i < j
                implies family(i).intersection(family(j)) = Set[K].empty_set
        }
        if range_indexed_union(k, family).intersection(family(k)).contains(x) {
            intersection_contains_eq(range_indexed_union(k, family), family(k), x)
            range_indexed_union(k, family).contains(x)
            family(k).contains(x)
            range_indexed_union_contains_witness(k, family, x)
            exists(i: Nat) { i < k and family(i).contains(x) }
            let i: Nat satisfy { i < k and family(i).contains(x) }
            lt_imp_lt_suc(i, k)
            i < k.suc
            lt_suc(k)
            k < k.suc
            family(i).intersection(family(k)) = Set[K].empty_set
            intersection_contains_eq(family(i), family(k), x)
            family(i).intersection(family(k)).contains(x)
            set_eq_contains_at(family(i).intersection(family(k)), Set[K].empty_set, x)
            family(i).intersection(family(k)).contains(x) =
                Set[K].empty_set.contains(x)
            Set[K].empty_set.contains(x) = false
            false
        }
        not range_indexed_union(k, family).intersection(family(k)).contains(x)
    }
}

/// If the members below `k.suc` are pairwise disjoint, the union of the first
/// `k` members is disjoint from the next member.
theorem range_indexed_union_disjoint_from_last[K](family: Nat -> Set[K], k: Nat) {
    pairwise_disjoint(family, k.suc)
        implies range_indexed_union(k, family).intersection(family(k)) = Set[K].empty_set
} by {
    if pairwise_disjoint(family, k.suc) {
        forall(x: K) {
            range_indexed_union_disjoint_last_contains(family, k, x)
            not range_indexed_union(k, family).intersection(family(k)).contains(x)
            empty_set_contains_eq[K](x)
            range_indexed_union(k, family).intersection(family(k)).contains(x) =
                Set[K].empty_set.contains(x)
        }
        set_ext(range_indexed_union(k, family).intersection(family(k)), Set[K].empty_set)
    }
}

/// Finite additivity for disjoint measurable families, quantified over the
/// bound: the measure of the union of the first `n` members equals the
/// partial sum of their measures.
theorem caratheodory_measurable_finite_disjoint_add_all[K](m: OuterMeasure[K],
        family: Nat -> Set[K]) {
    forall(n: Nat) {
        measurable_range(m, family, n) and pairwise_disjoint(family, n)
            implies m.measure(range_indexed_union(n, family)) = range_measure(m, family, n)
    }
} by {
    define p(k: Nat) -> Bool {
        measurable_range(m, family, k) and pairwise_disjoint(family, k)
            implies m.measure(range_indexed_union(k, family)) = range_measure(m, family, k)
    }
    if measurable_range(m, family, Nat.0) and pairwise_disjoint(family, Nat.0) {
        range_indexed_union_zero(family)
        range_indexed_union(Nat.0, family) = Set[K].empty_set
        m.measure(range_indexed_union(Nat.0, family)) = m.measure(Set[K].empty_set)
        outer_measure_empty(m)
        m.measure(Set[K].empty_set) = Real.0
        m.measure(range_indexed_union(Nat.0, family)) = Real.0
        range_measure_zero(m, family)
        range_measure(m, family, Nat.0) = Real.0
        m.measure(range_indexed_union(Nat.0, family)) = range_measure(m, family, Nat.0)
    }
    p(Nat.0)
    forall(k: Nat) {
        if p(k) {
            p(k) = (measurable_range(m, family, k) and pairwise_disjoint(family, k)
                implies m.measure(range_indexed_union(k, family)) = range_measure(m, family, k))
            if measurable_range(m, family, k.suc) and pairwise_disjoint(family, k.suc) {
                measurable_range(m, family, k.suc) =
                    forall(i: Nat) { i < k.suc implies caratheodory_measurable(m, family(i)) }
                pairwise_disjoint(family, k.suc) = forall(i0: Nat, j0: Nat) {
                    i0 < k.suc and j0 < k.suc and i0 < j0
                        implies family(i0).intersection(family(j0)) = Set[K].empty_set
                }
                measurable_range(m, family, k) =
                    forall(i: Nat) { i < k implies caratheodory_measurable(m, family(i)) }
                pairwise_disjoint(family, k) = forall(i0: Nat, j0: Nat) {
                    i0 < k and j0 < k and i0 < j0
                        implies family(i0).intersection(family(j0)) = Set[K].empty_set
                }
                forall(i: Nat) {
                    if i < k {
                        lt_imp_lt_suc(i, k)
                        i < k.suc
                        caratheodory_measurable(m, family(i))
                    }
                }
                measurable_range(m, family, k)
                forall(i0: Nat, j0: Nat) {
                    if i0 < k and j0 < k and i0 < j0 {
                        lt_imp_lt_suc(i0, k)
                        i0 < k.suc
                        lt_imp_lt_suc(j0, k)
                        j0 < k.suc
                        family(i0).intersection(family(j0)) = Set[K].empty_set
                    }
                }
                pairwise_disjoint(family, k)
                m.measure(range_indexed_union(k, family)) = range_measure(m, family, k)
                range_indexed_union_disjoint_from_last(family, k)
                pairwise_disjoint(family, k.suc)
                range_indexed_union(k, family).intersection(family(k)) = Set[K].empty_set
                lt_suc(k)
                k < k.suc
                caratheodory_measurable(m, family(k))
                caratheodory_measurable_disjoint_add_right(m, range_indexed_union(k, family), family(k))
                m.measure(range_indexed_union(k, family).union(family(k))) =
                    m.measure(range_indexed_union(k, family)) + m.measure(family(k))
                range_indexed_union_suc(family, k)
                range_indexed_union(k.suc, family) = range_indexed_union(k, family).union(family(k))
                m.measure(range_indexed_union(k.suc, family)) =
                    m.measure(range_indexed_union(k, family)) + m.measure(family(k))
                m.measure(range_indexed_union(k.suc, family)) =
                    range_measure(m, family, k) + m.measure(family(k))
                measure_seq(m, family, k) = m.measure(family(k))
                range_measure(m, family, k) + m.measure(family(k)) =
                    range_measure(m, family, k) + measure_seq(m, family, k)
                m.measure(range_indexed_union(k.suc, family)) =
                    range_measure(m, family, k) + measure_seq(m, family, k)
                range_measure_suc(m, family, k)
                range_measure(m, family, k.suc) =
                    range_measure(m, family, k) + measure_seq(m, family, k)
                m.measure(range_indexed_union(k.suc, family)) = range_measure(m, family, k.suc)
            }
            p(k.suc)
        }
    }
    p(Nat.0) and forall(k: Nat) { p(k) implies p(k.suc) }
    alt_induction(p)
    forall(k: Nat) { p(k) }
    forall(n: Nat) {
        (measurable_range(m, family, n) and pairwise_disjoint(family, n)
            implies m.measure(range_indexed_union(n, family)) = range_measure(m, family, n))
    }
}

/// Finite additivity for disjoint measurable families at a fixed bound.
theorem caratheodory_measurable_finite_disjoint_add[K](m: OuterMeasure[K],
        family: Nat -> Set[K], n: Nat) {
    measurable_range(m, family, n) and pairwise_disjoint(family, n)
        implies m.measure(range_indexed_union(n, family)) = range_measure(m, family, n)
} by {
    caratheodory_measurable_finite_disjoint_add_all(m, family)
    (measurable_range(m, family, n) and pairwise_disjoint(family, n)
        implies m.measure(range_indexed_union(n, family)) = range_measure(m, family, n))
}
