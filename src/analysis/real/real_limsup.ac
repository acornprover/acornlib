from nat import Nat
from real import Real, is_upper_bound, is_lower_bound, is_increasing, converges, limit,
    monotone_convergence_principle, increasing_convergent_bounded_by_limit, lte_trans,
    lte_add_right
from analysis.real.real_tail_supremum import tail_set, tail_sup, tail_sup_antitone, tail_sup_bounds_terms,
    tail_sup_is_least, is_bounded_above, is_bounded_above_intro

/// Negation reverses order.
///
/// `src/real/` has additive monotonicity but no statement of this, which every step below needs
/// because the tail suprema have to be flipped to be increasing.
theorem neg_reverses_lte(x: Real, y: Real) {
    x <= y implies -y <= -x
} by {
    if x <= y {
        lte_add_right(x, y, -x + -y)
        x + (-x + -y) <= y + (-x + -y)
        (x + (-x + -y) = (x + -x) + -y)
        (x + -x = Real.0)
        (Real.0 + -y = -y)
        (x + (-x + -y) = -y)
        (y + (-x + -y) = (y + -y) + -x)
        (y + -y = Real.0)
        (Real.0 + -x = -x)
        (y + (-x + -y) = -x)
        -y <= -x
    }
}

/// True if some real is below every term of the sequence.
define is_bounded_below(a: Nat -> Real) -> Bool {
    exists(b: Real) {
        is_lower_bound(a, b)
    }
}

/// A bound witnesses boundedness below.
theorem is_bounded_below_intro(a: Nat -> Real, b: Real) {
    is_lower_bound(a, b) implies is_bounded_below(a)
} by {
    if is_lower_bound(a, b) {
        exists(c: Real) { is_lower_bound(a, c) }
        (is_bounded_below(a) = exists(c: Real) { is_lower_bound(a, c) })
        is_bounded_below(a)
    }
}

/// Boundedness below yields a bound.
theorem is_bounded_below_witness(a: Nat -> Real) {
    is_bounded_below(a) implies exists(b: Real) { is_lower_bound(a, b) }
} by {
    if is_bounded_below(a) {
        (is_bounded_below(a) = exists(c: Real) { is_lower_bound(a, c) })
        exists(c: Real) { is_lower_bound(a, c) }
    }
}

/// A lower bound on the sequence is a lower bound on every tail supremum.
theorem tail_sup_above_lower_bound(a: Nat -> Real, b: Real, n: Nat) {
    is_bounded_above(a) and is_lower_bound(a, b) implies b <= tail_sup(a, n)
} by {
    if is_bounded_above(a) and is_lower_bound(a, b) {
        (is_lower_bound(a, b) = forall(k: Nat) { b <= a(k) })
        b <= a(n)
        n <= n
        tail_sup_bounds_terms(a, n, n)
        a(n) <= tail_sup(a, n)
        lte_trans(b, a(n), tail_sup(a, n))
        b <= tail_sup(a, n)
    }
}

/// The negated sequence of tail suprema.
///
/// Negated so that the antitone sequence of tail suprema becomes an increasing one, which is the
/// shape the monotone convergence principle is stated for. `src/real/` has no dual of that
/// principle and no existence theorem for infima, so this is the route to the limit superior
/// that needs nothing new.
define neg_tail_sup(a: Nat -> Real) -> (Nat -> Real) {
    function(n: Nat) {
        -tail_sup(a, n)
    }
}

/// The negated tail suprema increase.
theorem neg_tail_sup_increasing(a: Nat -> Real) {
    is_bounded_above(a) implies is_increasing(neg_tail_sup(a))
} by {
    if is_bounded_above(a) {
        forall(n: Nat) {
            n <= n.suc
            tail_sup_antitone(a, n, n.suc)
            tail_sup(a, n.suc) <= tail_sup(a, n)
            neg_reverses_lte(tail_sup(a, n.suc), tail_sup(a, n))
            -tail_sup(a, n) <= -tail_sup(a, n.suc)
            (neg_tail_sup(a)(n) = -tail_sup(a, n))
            (neg_tail_sup(a)(n.suc) = -tail_sup(a, n.suc))
            neg_tail_sup(a)(n) <= neg_tail_sup(a)(n.suc)
        }
        (is_increasing(neg_tail_sup(a)) = forall(n: Nat) {
            neg_tail_sup(a)(n) <= neg_tail_sup(a)(n.suc)
        })
        is_increasing(neg_tail_sup(a))
    }
}

/// The negated tail suprema are bounded above by the negation of any lower bound.
theorem neg_tail_sup_bounded(a: Nat -> Real, b: Real) {
    is_bounded_above(a) and is_lower_bound(a, b)
        implies is_upper_bound(neg_tail_sup(a), -b)
} by {
    if is_bounded_above(a) and is_lower_bound(a, b) {
        forall(n: Nat) {
            tail_sup_above_lower_bound(a, b, n)
            b <= tail_sup(a, n)
            neg_reverses_lte(b, tail_sup(a, n))
            -tail_sup(a, n) <= -b
            (neg_tail_sup(a)(n) = -tail_sup(a, n))
            neg_tail_sup(a)(n) <= -b
        }
        (is_upper_bound(neg_tail_sup(a), -b) = forall(n: Nat) {
            neg_tail_sup(a)(n) <= -b
        })
        is_upper_bound(neg_tail_sup(a), -b)
    }
}

/// The negated tail suprema converge for a bounded sequence.
theorem neg_tail_sup_converges(a: Nat -> Real) {
    is_bounded_above(a) and is_bounded_below(a) implies converges(neg_tail_sup(a))
} by {
    if is_bounded_above(a) and is_bounded_below(a) {
        is_bounded_below_witness(a)
        exists(b: Real) { is_lower_bound(a, b) }
        let (b: Real) satisfy {
            is_lower_bound(a, b)
        }
        neg_tail_sup_increasing(a)
        is_increasing(neg_tail_sup(a))
        neg_tail_sup_bounded(a, b)
        is_upper_bound(neg_tail_sup(a), -b)
        monotone_convergence_principle(neg_tail_sup(a), -b)
        converges(neg_tail_sup(a))
    }
}

/// The limit superior of a sequence.
///
/// The eventual least bound: the tail suprema decrease and settle, and this is where they
/// settle. Written through the negated sequence because that is the direction the monotone
/// convergence principle runs; for a sequence that is not bounded on both sides the value is
/// whatever `limit` defaults to and means nothing.
define limsup(a: Nat -> Real) -> Real {
    -limit(neg_tail_sup(a))
}

/// The limit superior is at most every tail supremum.
///
/// The tail suprema decrease to it, so none of them is below it.
theorem limsup_le_tail_sup(a: Nat -> Real, n: Nat) {
    is_bounded_above(a) and is_bounded_below(a) implies limsup(a) <= tail_sup(a, n)
} by {
    if is_bounded_above(a) and is_bounded_below(a) {
        neg_tail_sup_converges(a)
        converges(neg_tail_sup(a))
        neg_tail_sup_increasing(a)
        is_increasing(neg_tail_sup(a))
        increasing_convergent_bounded_by_limit(neg_tail_sup(a))
        is_upper_bound(neg_tail_sup(a), limit(neg_tail_sup(a)))
        (is_upper_bound(neg_tail_sup(a), limit(neg_tail_sup(a))) = forall(k: Nat) {
            neg_tail_sup(a)(k) <= limit(neg_tail_sup(a))
        })
        neg_tail_sup(a)(n) <= limit(neg_tail_sup(a))
        (neg_tail_sup(a)(n) = -tail_sup(a, n))
        -tail_sup(a, n) <= limit(neg_tail_sup(a))
        neg_reverses_lte(-tail_sup(a, n), limit(neg_tail_sup(a)))
        -limit(neg_tail_sup(a)) <= -(-tail_sup(a, n))
        (-(-tail_sup(a, n)) = tail_sup(a, n))
        -limit(neg_tail_sup(a)) <= tail_sup(a, n)
        (limsup(a) = -limit(neg_tail_sup(a)))
        limsup(a) <= tail_sup(a, n)
    }
}
