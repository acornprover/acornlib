from nat import Nat
from real import Real, neg_neg, neg_distrib
from data.basic.functions import function_extensionality
from algebra.add_comm_monoid_rearrange import add_swap_inner
from analysis.real.real_tail_supremum import is_bounded_above
from analysis.real.real_limsup import limsup, is_bounded_below
from analysis.real.real_liminf import liminf
from analysis.real.real_one_minus_seq import one_minus_seq, one_minus_bounded_above,
    one_minus_bounded_below, limsup_one_minus

/// Reflection in one undoes itself.
theorem one_minus_one_minus(x: Real) {
    Real.1 - (Real.1 - x) = x
} by {
    (Real.1 - x = Real.1 + -x)
    neg_distrib(Real.1, -x)
    (-(Real.1 + -x) = -Real.1 + -(-x))
    neg_neg(x)
    (-(-x) = x)
    (-(Real.1 - x) = -Real.1 + x)
    (Real.1 - (Real.1 - x) = Real.1 + (-Real.1 + x))
    add_swap_inner[Real](Real.1, Real.0, -Real.1, x)
    ((Real.1 + Real.0) + (-Real.1 + x) = (Real.1 + -Real.1) + (Real.0 + x))
    (Real.1 + Real.0 = Real.1)
    (Real.1 + -Real.1 = Real.0)
    (Real.0 + x = x)
    (Real.1 + (-Real.1 + x) = x)
    Real.1 - (Real.1 - x) = x
}

/// A number and its reflection in one add to one.
theorem add_one_minus(x: Real) {
    x + (Real.1 - x) = Real.1
} by {
    (Real.1 - x = Real.1 + -x)
    ((Real.1 + -x) + x = Real.1 + (-x + x))
    (-x + x = Real.0)
    (Real.1 + Real.0 = Real.1)
    ((Real.1 - x) + x = Real.1)
    (x + (Real.1 - x) = (Real.1 - x) + x)
    x + (Real.1 - x) = Real.1
}

/// Reflecting twice in one restores the sequence.
///
/// Reflection in one is an involution, which is what lets the limit inferior of a reflection be
/// read off the statement about the limit superior rather than proved again from the tails.
theorem one_minus_seq_involution(a: Nat -> Real) {
    one_minus_seq(one_minus_seq(a)) = a
} by {
    forall(n: Nat) {
        (one_minus_seq(a)(n) = Real.1 - a(n))
        (one_minus_seq(one_minus_seq(a))(n) = Real.1 - one_minus_seq(a)(n))
        one_minus_one_minus(a(n))
        (Real.1 - (Real.1 - a(n)) = a(n))
        one_minus_seq(one_minus_seq(a))(n) = a(n)
    }
    function_extensionality[Nat, Real](one_minus_seq(one_minus_seq(a)), a)
    one_minus_seq(one_minus_seq(a)) = a
}

/// The limit inferior of a reflected sequence is the reflection of the limit superior.
///
/// The dual of `limsup_one_minus`, obtained by reflecting that statement at the reflected
/// sequence: reflection is an involution, so the doubly reflected limit superior is the
/// original one.
theorem liminf_one_minus(a: Nat -> Real) {
    is_bounded_above(a) and is_bounded_below(a)
        implies liminf(one_minus_seq(a)) = Real.1 - limsup(a)
} by {
    if is_bounded_above(a) and is_bounded_below(a) {
        one_minus_bounded_above(a)
        is_bounded_above(one_minus_seq(a))
        one_minus_bounded_below(a)
        is_bounded_below(one_minus_seq(a))
        limsup_one_minus(one_minus_seq(a))
        (limsup(one_minus_seq(one_minus_seq(a)))
            = Real.1 - liminf(one_minus_seq(a)))
        one_minus_seq_involution(a)
        (one_minus_seq(one_minus_seq(a)) = a)
        limsup(a) = Real.1 - liminf(one_minus_seq(a))
        one_minus_one_minus(liminf(one_minus_seq(a)))
        (Real.1 - (Real.1 - liminf(one_minus_seq(a))) = liminf(one_minus_seq(a)))
        Real.1 - limsup(a) = liminf(one_minus_seq(a))
        liminf(one_minus_seq(a)) = Real.1 - limsup(a)
    }
}
