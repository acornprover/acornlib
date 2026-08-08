from nat import Nat, div_mod_decomp, mod_lt, lte_antisymm, lt_and_lte, lt_add_left
from int import Int, lte_from_nat, mul_nat_from_nat_left, mul_nat_from_nat_right,
    from_nat_pos, lt_from_nat
from rat import Rat, from_int_num, lt_from_int
from real import Real, floor, mul_le_mul_pos_left, mul_le_mul_pos_right,
    div_le_of_mul_le, mul_div_cancel, lt_some_int_cancel, lte_lt_trans, from_rat_pos, from_rat_maintains_lt
from analysis.real.real_base_digits import real_from_int_lte, real_from_int_mul, floor_is_greatest,
    int_lt_imp_add_one_lte

numerals Int

/// The embedding of the naturals into the integers preserves products.
theorem int_from_nat_mul(a: Nat, b: Nat) {
    Int.from_nat(a) * Int.from_nat(b) = Int.from_nat(a * b)
} by {
    mul_nat_from_nat_right(Int.from_nat(a), b)
    (Int.from_nat(a).mul_nat(b) = Int.from_nat(a) * Int.from_nat(b))
    mul_nat_from_nat_left(a, b)
    (Int.from_nat(a).mul_nat(b) = Int.from_nat(a * b))
    Int.from_nat(a) * Int.from_nat(b) = Int.from_nat(a * b)
}

/// The embedding of a positive integer into the reals is positive.
theorem real_from_int_pos(n: Int) {
    n.is_positive implies Real.from_int(n) > Real.0
} by {
    if n.is_positive {
        from_int_num(n)
        (Rat.from_int(n).num = n)
        (Rat.from_int(n).is_positive = Rat.from_int(n).num.is_positive)
        Rat.from_int(n).is_positive
        from_rat_pos(Rat.from_int(n))
        Real.from_rat(Rat.from_int(n)).is_positive
        (Real.from_int(n) = Real.from_rat(Rat.from_int(n)))
        Real.from_int(n).is_positive
        Real.from_int(n) > Real.0
    }
}

/// A quotient below a bound follows from the product being below it.
///
/// The mirror of `div_le_of_mul_le`, in the strict direction. `src/real/` supplies one of the two
/// and every floor-of-a-quotient argument needs both.
theorem lt_div_of_lt_mul(a: Real, b: Real, c: Real) {
    c < b * a and b > Real.0 implies c / b < a
} by {
    if c < b * a and b > Real.0 {
        if a <= c / b {
            mul_le_mul_pos_left(a, c / b, b)
            (b * a <= b * (c / b))
            b != Real.0
            mul_div_cancel(c, b)
            (b * (c / b) = c)
            b * a <= c
            false
        }
        c / b < a
    }
}

/// The embedding of the integers into the reals preserves the strict order.
theorem real_from_int_lt(m: Int, n: Int) {
    m < n implies Real.from_int(m) < Real.from_int(n)
} by {
    if m < n {
        lt_from_int(m, n)
        (Rat.from_int(m) < Rat.from_int(n))
        from_rat_maintains_lt(Rat.from_int(m), Rat.from_int(n))
        (Real.from_rat(Rat.from_int(m)) < Real.from_rat(Rat.from_int(n)))
        (Real.from_int(m) = Real.from_rat(Rat.from_int(m)))
        (Real.from_int(n) = Real.from_rat(Rat.from_int(n)))
        Real.from_int(m) < Real.from_int(n)
    }
}

/// Below one more means at most, over the integers.
theorem int_lt_add_one_imp_lte(a: Int, b: Int) {
    a < b + Int.1 implies a <= b
} by {
    if a < b + Int.1 {
        int_lt_imp_add_one_lte(a, b + Int.1)
        (a + Int.1 <= b + Int.1)
        ((b + Int.1 - (a + Int.1)).is_positive or a + Int.1 = b + Int.1)
        (b + Int.1 - (a + Int.1) = b - a)
        if a + Int.1 = b + Int.1 {
            a = b
        }
        ((b - a).is_positive or a = b)
        a <= b
    }
}

/// The floor of a quotient of naturals is their quotient.
///
/// The bridge between the real floor and natural division, which nothing in the library states.
/// The division algorithm puts the quotient below the ratio and its successor above, and the floor
/// is characterised by exactly that.
theorem floor_nat_div(a: Nat, b: Nat) {
    Nat.0 < b implies floor(Real.from_int(Int.from_nat(a)) / Real.from_int(Int.from_nat(b)))
        = Int.from_nat(a.div(b))
} by {
    if Nat.0 < b {
        b != Nat.0
        div_mod_decomp(a, b)
        (a.div(b) * b + a.mod(b) = a)
        mod_lt(a, b)
        a.mod(b) < b
        (a.div(b) * b <= a)
        lte_from_nat(a.div(b) * b, a)
        (Int.from_nat(a.div(b) * b) <= Int.from_nat(a))
        int_from_nat_mul(a.div(b), b)
        (Int.from_nat(a.div(b)) * Int.from_nat(b) = Int.from_nat(a.div(b) * b))
        real_from_int_lte(Int.from_nat(a.div(b) * b), Int.from_nat(a))
        (Real.from_int(Int.from_nat(a.div(b) * b))
            <= Real.from_int(Int.from_nat(a)))
        real_from_int_mul(Int.from_nat(a.div(b)), Int.from_nat(b))
        (Real.from_int(Int.from_nat(a.div(b))) * Real.from_int(Int.from_nat(b))
            = Real.from_int(Int.from_nat(a.div(b)) * Int.from_nat(b)))
        (Real.from_int(Int.from_nat(a.div(b))) * Real.from_int(Int.from_nat(b))
            <= Real.from_int(Int.from_nat(a)))
        from_nat_pos(b)
        Int.from_nat(b).is_positive
        real_from_int_pos(Int.from_nat(b))
        Real.from_int(Int.from_nat(b)) > Real.0
        div_le_of_mul_le(Real.from_int(Int.from_nat(a.div(b))),
            Real.from_int(Int.from_nat(b)), Real.from_int(Int.from_nat(a)))
        (Real.from_int(Int.from_nat(a.div(b)))
            <= Real.from_int(Int.from_nat(a)) / Real.from_int(Int.from_nat(b)))
        floor_is_greatest(Real.from_int(Int.from_nat(a)) / Real.from_int(Int.from_nat(b)),
            Int.from_nat(a.div(b)))
        (Int.from_nat(a.div(b))
            <= floor(Real.from_int(Int.from_nat(a)) / Real.from_int(Int.from_nat(b))))
        ((a.div(b) + Nat.1) * b = a.div(b) * b + b)
        lt_add_left(a.div(b) * b, a.mod(b), b)
        (a.div(b) * b + a.mod(b) < a.div(b) * b + b)
        (a < (a.div(b) + Nat.1) * b)
        lt_from_nat(a, (a.div(b) + Nat.1) * b)
        (Int.from_nat(a) < Int.from_nat((a.div(b) + Nat.1) * b))
        int_from_nat_mul(a.div(b) + Nat.1, b)
        (Int.from_nat(a.div(b) + Nat.1) * Int.from_nat(b)
            = Int.from_nat((a.div(b) + Nat.1) * b))
        real_from_int_lt(Int.from_nat(a), Int.from_nat((a.div(b) + Nat.1) * b))
        (Real.from_int(Int.from_nat(a))
            < Real.from_int(Int.from_nat((a.div(b) + Nat.1) * b)))
        real_from_int_mul(Int.from_nat(b), Int.from_nat(a.div(b) + Nat.1))
        (Real.from_int(Int.from_nat(b)) * Real.from_int(Int.from_nat(a.div(b) + Nat.1))
            = Real.from_int(Int.from_nat(b) * Int.from_nat(a.div(b) + Nat.1)))
        (Int.from_nat(b) * Int.from_nat(a.div(b) + Nat.1)
            = Int.from_nat(a.div(b) + Nat.1) * Int.from_nat(b))
        (Real.from_int(Int.from_nat(b)) * Real.from_int(Int.from_nat(a.div(b) + Nat.1))
            = Real.from_int(Int.from_nat((a.div(b) + Nat.1) * b)))
        (Real.from_int(Int.from_nat(a))
            < Real.from_int(Int.from_nat(b))
                * Real.from_int(Int.from_nat(a.div(b) + Nat.1)))
        lt_div_of_lt_mul(Real.from_int(Int.from_nat(a.div(b) + Nat.1)),
            Real.from_int(Int.from_nat(b)), Real.from_int(Int.from_nat(a)))
        (Real.from_int(Int.from_nat(a)) / Real.from_int(Int.from_nat(b))
            < Real.from_int(Int.from_nat(a.div(b) + Nat.1)))
        (Real.from_int(floor(Real.from_int(Int.from_nat(a))
            / Real.from_int(Int.from_nat(b))))
            <= Real.from_int(Int.from_nat(a)) / Real.from_int(Int.from_nat(b)))
        lte_lt_trans(Real.from_int(floor(Real.from_int(Int.from_nat(a))
            / Real.from_int(Int.from_nat(b)))),
            Real.from_int(Int.from_nat(a)) / Real.from_int(Int.from_nat(b)),
            Real.from_int(Int.from_nat(a.div(b) + Nat.1)))
        lt_some_int_cancel(floor(Real.from_int(Int.from_nat(a))
            / Real.from_int(Int.from_nat(b))), Int.from_nat(a.div(b) + Nat.1))
        (floor(Real.from_int(Int.from_nat(a)) / Real.from_int(Int.from_nat(b)))
            < Int.from_nat(a.div(b) + Nat.1))
        (Int.from_nat(a.div(b) + Nat.1) = Int.from_nat(a.div(b)) + Int.1)
        (floor(Real.from_int(Int.from_nat(a)) / Real.from_int(Int.from_nat(b)))
            < Int.from_nat(a.div(b)) + Int.1)
        int_lt_add_one_imp_lte(floor(Real.from_int(Int.from_nat(a))
            / Real.from_int(Int.from_nat(b))), Int.from_nat(a.div(b)))
        (floor(Real.from_int(Int.from_nat(a)) / Real.from_int(Int.from_nat(b)))
            <= Int.from_nat(a.div(b)))
        (floor(Real.from_int(Int.from_nat(a)) / Real.from_int(Int.from_nat(b)))
            = Int.from_nat(a.div(b)))
    }
}
