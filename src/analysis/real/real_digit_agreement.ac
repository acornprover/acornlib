from nat import Nat, add_sub, add_imp_sub, pow_zero, pow_one, pow_add, lt_and_lte
from int import Int, add_from_nat
from real import Real, floor, mul_div_cancel, mul_left_cancel
from analysis.real.real_base_digits import real_digit, real_from_int_mul, real_scaled_suc
from analysis.real.real_floor_div import int_from_nat_mul, floor_nat_div, real_from_int_pos
from data.nat.nat_div_div import nat_digit, nat_pow_pos, nat_div_pow_suc, nat_digit_eq_div_sub

numerals Nat

/// Adding then subtracting the same integer on the left changes nothing.
theorem int_add_sub_cancel_left(x: Int, y: Int) {
    (x + y) - x = y
}

/// Cancelling one factor of a denominator.
///
/// The division identity a scaled digit needs: multiplying by a factor of the denominator leaves
/// the quotient by what is left of it.
theorem real_mul_div_factor(x: Real, c: Real, d: Real) {
    c != Real.0 and d != Real.0 implies c * (x / (c * d)) = x / d
} by {
    if c != Real.0 and d != Real.0 {
        c * d != Real.0
        mul_div_cancel(x, c * d)
        ((c * d) * (x / (c * d)) = x)
        (d * (c * (x / (c * d))) = (c * d) * (x / (c * d)))
        (x = d * (c * (x / (c * d))))
        mul_left_cancel(x, d, c * (x / (c * d)))
        (x / d = c * (x / (c * d)))
        c * (x / (c * d)) = x / d
    }
}

/// The embedding of a natural into the reals commutes with powers.
theorem real_from_nat_pow(b: Nat, k: Nat) {
    Real.from_int(Int.from_nat(b)).pow(k) = Real.from_int(Int.from_nat(b.pow(k)))
} by {
    define p(x: Nat) -> Bool {
        Real.from_int(Int.from_nat(b)).pow(x) = Real.from_int(Int.from_nat(b.pow(x)))
    }
    (Real.from_int(Int.from_nat(b)).pow(Nat.0) = Real.1)
    pow_zero[Nat](b)
    (b.pow(Nat.0) = Nat.1)
    (Int.from_nat(Nat.1) = Int.1)
    (Real.from_int(Int.1) = Real.1)
    p(Nat.0)
    forall(x: Nat) {
        if p(x) {
            (Real.from_int(Int.from_nat(b)).pow(x)
                = Real.from_int(Int.from_nat(b.pow(x))))
            (Real.from_int(Int.from_nat(b)).pow(x.suc)
                = Real.from_int(Int.from_nat(b)).pow(x)
                    * Real.from_int(Int.from_nat(b)))
            real_from_int_mul(Int.from_nat(b.pow(x)), Int.from_nat(b))
            (Real.from_int(Int.from_nat(b.pow(x))) * Real.from_int(Int.from_nat(b))
                = Real.from_int(Int.from_nat(b.pow(x)) * Int.from_nat(b)))
            int_from_nat_mul(b.pow(x), b)
            (Int.from_nat(b.pow(x)) * Int.from_nat(b) = Int.from_nat(b.pow(x) * b))
            pow_one[Nat](b)
            (b.pow(Nat.1) = b)
            pow_add[Nat](b, x, Nat.1)
            (b.pow(x + Nat.1) = b.pow(x) * b.pow(Nat.1))
            (x + Nat.1 = x.suc)
            (b.pow(x.suc) = b.pow(x) * b)
            (Real.from_int(Int.from_nat(b)).pow(x.suc)
                = Real.from_int(Int.from_nat(b.pow(x.suc))))
            p(x.suc)
        }
        (p(x) implies p(x.suc))
    }
    p(Nat.0) and forall(x: Nat) {
        p(x) implies p(x.suc)
    }
    Nat.induction(p)
    p(k)
}

/// Scaling a fraction with a power denominator by a lower power lowers the denominator.
theorem real_scaled_fraction(b: Nat, n: Nat, m: Nat, k: Nat) {
    Nat.0 < b and k <= m implies
        Real.from_int(Int.from_nat(b)).pow(k)
            * (Real.from_int(Int.from_nat(n)) / Real.from_int(Int.from_nat(b.pow(m))))
        = Real.from_int(Int.from_nat(n)) / Real.from_int(Int.from_nat(b.pow(m - k)))
} by {
    if Nat.0 < b and k <= m {
        add_sub(m, k)
        ((m - k) + k = m)
        (k + (m - k) = m)
        pow_add[Nat](b, k, m - k)
        (b.pow(k + (m - k)) = b.pow(k) * b.pow(m - k))
        (b.pow(m) = b.pow(k) * b.pow(m - k))
        int_from_nat_mul(b.pow(k), b.pow(m - k))
        (Int.from_nat(b.pow(k)) * Int.from_nat(b.pow(m - k))
            = Int.from_nat(b.pow(k) * b.pow(m - k)))
        real_from_int_mul(Int.from_nat(b.pow(k)), Int.from_nat(b.pow(m - k)))
        (Real.from_int(Int.from_nat(b.pow(k)))
            * Real.from_int(Int.from_nat(b.pow(m - k)))
            = Real.from_int(Int.from_nat(b.pow(k) * b.pow(m - k))))
        (Real.from_int(Int.from_nat(b.pow(m)))
            = Real.from_int(Int.from_nat(b.pow(k)))
                * Real.from_int(Int.from_nat(b.pow(m - k))))
        nat_pow_pos(b, k)
        Nat.0 < b.pow(k)
        Int.from_nat(b.pow(k)).is_positive
        real_from_int_pos(Int.from_nat(b.pow(k)))
        Real.from_int(Int.from_nat(b.pow(k))) > Real.0
        Real.from_int(Int.from_nat(b.pow(k))) != Real.0
        nat_pow_pos(b, m - k)
        Nat.0 < b.pow(m - k)
        Int.from_nat(b.pow(m - k)).is_positive
        real_from_int_pos(Int.from_nat(b.pow(m - k)))
        Real.from_int(Int.from_nat(b.pow(m - k))) > Real.0
        Real.from_int(Int.from_nat(b.pow(m - k))) != Real.0
        real_mul_div_factor(Real.from_int(Int.from_nat(n)),
            Real.from_int(Int.from_nat(b.pow(k))),
            Real.from_int(Int.from_nat(b.pow(m - k))))
        real_from_nat_pow(b, k)
        (Real.from_int(Int.from_nat(b)).pow(k) = Real.from_int(Int.from_nat(b.pow(k))))
        (Real.from_int(Int.from_nat(b)).pow(k)
            * (Real.from_int(Int.from_nat(n)) / Real.from_int(Int.from_nat(b.pow(m))))
            = Real.from_int(Int.from_nat(n)) / Real.from_int(Int.from_nat(b.pow(m - k))))
    }
}

/// The scaled floor of a fraction with a power denominator is a natural quotient.
theorem floor_scaled_fraction(b: Nat, n: Nat, m: Nat, k: Nat) {
    Nat.0 < b and k <= m implies
        floor(Real.from_int(Int.from_nat(b)).pow(k)
            * (Real.from_int(Int.from_nat(n)) / Real.from_int(Int.from_nat(b.pow(m)))))
        = Int.from_nat(n.div(b.pow(m - k)))
} by {
    if Nat.0 < b and k <= m {
        real_scaled_fraction(b, n, m, k)
        (Real.from_int(Int.from_nat(b)).pow(k)
            * (Real.from_int(Int.from_nat(n)) / Real.from_int(Int.from_nat(b.pow(m))))
            = Real.from_int(Int.from_nat(n)) / Real.from_int(Int.from_nat(b.pow(m - k))))
        nat_pow_pos(b, m - k)
        Nat.0 < b.pow(m - k)
        floor_nat_div(n, b.pow(m - k))
        (floor(Real.from_int(Int.from_nat(n))
            / Real.from_int(Int.from_nat(b.pow(m - k))))
            = Int.from_nat(n.div(b.pow(m - k))))
        (floor(Real.from_int(Int.from_nat(b)).pow(k)
            * (Real.from_int(Int.from_nat(n)) / Real.from_int(Int.from_nat(b.pow(m)))))
            = Int.from_nat(n.div(b.pow(m - k))))
    }
}

/// The digits of a fraction with a power denominator are the digits of its numerator.
///
/// The agreement between the two expansions. Reading `n / b^m` after the point at place `k` is
/// reading `n` before the point at place `m - k - 1`: the real expansion runs outward from the
/// point and the natural one inward, so the places are counted in opposite directions.
theorem real_digit_eq_nat_digit(b: Nat, n: Nat, m: Nat, k: Nat) {
    Nat.0 < b and k.suc <= m implies
        real_digit(Int.from_nat(b),
            Real.from_int(Int.from_nat(n)) / Real.from_int(Int.from_nat(b.pow(m))), k)
        = Int.from_nat(nat_digit(b, n, m - k.suc))
} by {
    if Nat.0 < b and k.suc <= m {
        k < k.suc
        lt_and_lte(k, k.suc, m)
        k < m
        k <= m
        floor_scaled_fraction(b, n, m, k)
        (floor(Real.from_int(Int.from_nat(b)).pow(k)
            * (Real.from_int(Int.from_nat(n)) / Real.from_int(Int.from_nat(b.pow(m)))))
            = Int.from_nat(n.div(b.pow(m - k))))
        floor_scaled_fraction(b, n, m, k.suc)
        (floor(Real.from_int(Int.from_nat(b)).pow(k.suc)
            * (Real.from_int(Int.from_nat(n)) / Real.from_int(Int.from_nat(b.pow(m)))))
            = Int.from_nat(n.div(b.pow(m - k.suc))))
        (real_digit(Int.from_nat(b),
            Real.from_int(Int.from_nat(n)) / Real.from_int(Int.from_nat(b.pow(m))), k)
            = Int.from_nat(n.div(b.pow(m - k.suc)))
                - Int.from_nat(b) * Int.from_nat(n.div(b.pow(m - k))))
        add_sub(m, k.suc)
        ((m - k.suc) + k.suc = m)
        ((m - k.suc).suc + k = m)
        add_imp_sub(m - k.suc, k.suc, m)
        ((m - k.suc).suc = m - k)
        nat_div_pow_suc(b, n, m - k.suc)
        (n.div(b.pow((m - k.suc).suc)) = (n.div(b.pow(m - k.suc))).div(b))
        (n.div(b.pow(m - k)) = (n.div(b.pow(m - k.suc))).div(b))
        nat_digit_eq_div_sub(b, n, m - k.suc)
        ((n.div(b.pow(m - k.suc))).div(b) * b + nat_digit(b, n, m - k.suc)
            = n.div(b.pow(m - k.suc)))
        (n.div(b.pow(m - k)) * b + nat_digit(b, n, m - k.suc)
            = n.div(b.pow(m - k.suc)))
        add_from_nat(n.div(b.pow(m - k)) * b, nat_digit(b, n, m - k.suc))
        (Int.from_nat(n.div(b.pow(m - k)) * b)
            + Int.from_nat(nat_digit(b, n, m - k.suc))
            = Int.from_nat(n.div(b.pow(m - k.suc))))
        int_from_nat_mul(n.div(b.pow(m - k)), b)
        (Int.from_nat(n.div(b.pow(m - k))) * Int.from_nat(b)
            = Int.from_nat(n.div(b.pow(m - k)) * b))
        (Int.from_nat(b) * Int.from_nat(n.div(b.pow(m - k)))
            = Int.from_nat(n.div(b.pow(m - k)) * b))
        int_add_sub_cancel_left(Int.from_nat(n.div(b.pow(m - k)) * b),
            Int.from_nat(nat_digit(b, n, m - k.suc)))
        ((Int.from_nat(n.div(b.pow(m - k)) * b)
            + Int.from_nat(nat_digit(b, n, m - k.suc)))
            - Int.from_nat(n.div(b.pow(m - k)) * b)
            = Int.from_nat(nat_digit(b, n, m - k.suc)))
        (Int.from_nat(n.div(b.pow(m - k.suc))) - Int.from_nat(n.div(b.pow(m - k)) * b)
            = Int.from_nat(nat_digit(b, n, m - k.suc)))
        (Int.from_nat(n.div(b.pow(m - k.suc)))
            - Int.from_nat(b) * Int.from_nat(n.div(b.pow(m - k)))
            = Int.from_nat(nat_digit(b, n, m - k.suc)))
        (real_digit(Int.from_nat(b),
            Real.from_int(Int.from_nat(n)) / Real.from_int(Int.from_nat(b.pow(m))), k)
            = Int.from_nat(nat_digit(b, n, m - k.suc)))
    }
}
