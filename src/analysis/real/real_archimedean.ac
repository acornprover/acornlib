from nat import Nat, from_nat
from rat import Rat, lt_some_nat
from real import Real, from_rat_maintains_lt, from_nat_is_from_rat, rat_upper, lt_trans,
    div_lt_mul_pos

/// Every real is below some natural number.
///
/// The Archimedean property. `src/real/` states neither this nor anything equivalent, but both
/// halves are there: every real has a rational strictly above it, and the rationals are
/// Archimedean because they are built from integers.
theorem real_lt_some_nat(x: Real) {
    exists(n: Nat) {
        x < from_nat[Real](n)
    }
} by {
    Rat.1.is_positive
    rat_upper(x, Rat.1)
    exists(r: Rat) {
        not x.is_close(Real.from_rat(r), Real.from_rat(Rat.1)) and x < Real.from_rat(r)
    }
    let (r: Rat) satisfy {
        not x.is_close(Real.from_rat(r), Real.from_rat(Rat.1)) and x < Real.from_rat(r)
    }
    x < Real.from_rat(r)
    lt_some_nat(r)
    exists(m: Nat) { r < Rat.from_nat(m) }
    let (n: Nat) satisfy {
        r < Rat.from_nat(n)
    }
    from_rat_maintains_lt(r, Rat.from_nat(n))
    Real.from_rat(r) < Real.from_rat(Rat.from_nat(n))
    from_nat_is_from_rat(n)
    from_nat[Real](n) = Real.from_rat(Rat.from_nat(n))
    Real.from_rat(r) < from_nat[Real](n)
    lt_trans(x, Real.from_rat(r), from_nat[Real](n))
    x < from_nat[Real](n)
    exists(m: Nat) { x < from_nat[Real](m) }
}

/// Any real is below some natural multiple of any positive real.
///
/// The form an approximation argument uses: given a target and a size, enough copies of the size
/// exceed the target. Stated with the multiple on the left of the product so that it composes
/// with the division lemmas of `src/real/`.
theorem lt_nat_multiple(x: Real, eps: Real) {
    eps > Real.0 implies exists(n: Nat) {
        x < eps * from_nat[Real](n)
    }
} by {
    if eps > Real.0 {
        real_lt_some_nat(x / eps)
        exists(m: Nat) { x / eps < from_nat[Real](m) }
        let (n: Nat) satisfy {
            x / eps < from_nat[Real](n)
        }
        div_lt_mul_pos(x, eps, from_nat[Real](n))
        x < eps * from_nat[Real](n)
        exists(m: Nat) { x < eps * from_nat[Real](m) }
    }
}
