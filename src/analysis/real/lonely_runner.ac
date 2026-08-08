/// The lonely runner conjecture, small cases.
///
/// Runner 0 is the lonely runner, stationary at the origin. The remaining runners move
/// at distinct positive integer speeds; a runner is lonely when its circular distance
/// from the origin is at least `1/(n+1)` of the circumference, where `n` is the total
/// number of runners. The conjecture is open in general (it is known for up to seven
/// moving runners). This file proves the cases with up to three moving runners, for the
/// smallest concrete speed tuples, and states the general conjecture.
from nat import Nat, from_nat, lt_add_suc, lt_imp_lte_suc, lte_and_lt, lt_and_lte,
    add_sub, add_comm, sub_pos, lte_ref
from int import Int
from order import lt_imp_lte, lt_not_ref, lt_imp_ne, lte_min_of_bounds, lt_iff_lte_and_ne
from real import Real, floor, lt_add_one, lt_lte_trans,
    mul_one_left, mul_div_cancel, mul_one_over, div_cancel_common,
    sub_cancels, mul_sub_distrib_left, mul_inverse
from ordered_field import mul_lt_mul_of_pos_right, inverse_of_positive_is_positive
from algebra.field.field import mul_not_zero
from analysis.real.real_base_digits import real_from_int_lte, floor_is_greatest, int_lt_imp_add_one_lte
from analysis.real.real_from_nat_order import from_nat_real_lte, div_le_div_pos, from_nat_real_nonnegative
from data.basic.logic import exists_intro

numerals Real
numerals Nat

/// The fractional part of a real number: `x - floor(x)`, an element of [0, 1).
define frac(x: Real) -> Real {
    x - Real.from_int(floor(x))
}

/// The circular distance on the unit circle from `x` to the origin.
define circular_distance(x: Real) -> Real {
    frac(x).min(Real.1 - frac(x))
}

// ---- fractional part and circular distance on the unit interval ----

/// The floor of a number in [0, 1) is zero.
theorem floor_zero_of_unit_interval(x: Real) {
    Real.0 <= x and x < Real.1 implies floor(x) = Int.0
} by {
    if Real.0 <= x and x < Real.1 {
        (Real.from_int(Int.0) = Real.0)
        (Real.from_int(Int.0) <= x)
        floor_is_greatest(x, Int.0)
        Int.0 <= floor(x)
        if Int.1 <= floor(x) {
            real_from_int_lte(Int.1, floor(x))
            (Real.from_int(Int.1) <= Real.from_int(floor(x)))
            (Real.from_int(Int.1) = Real.1)
            Real.1 <= Real.from_int(floor(x))
            lt_lte_trans(x, Real.1, Real.from_int(floor(x)))
            x < Real.from_int(floor(x))
            (Real.from_int(floor(x)) <= x)
            lt_lte_trans(x, Real.from_int(floor(x)), x)
            x < x
            lt_not_ref(x)
            false
        }
        not (Int.1 <= floor(x))
        if floor(x) != Int.0 {
            lt_iff_lte_and_ne(Int.0, floor(x))
            (Int.0 < floor(x)) = (Int.0 <= floor(x) and Int.0 != floor(x))
            Int.0 < floor(x)
            int_lt_imp_add_one_lte(Int.0, floor(x))
            (Int.0 + Int.1 <= floor(x))
            (Int.0 + Int.1 = Int.1)
            Int.1 <= floor(x)
            false
        }
        floor(x) = Int.0
    }
}

/// The fractional part of a number in [0, 1) is the number itself.
theorem frac_of_unit_interval(x: Real) {
    Real.0 <= x and x < Real.1 implies frac(x) = x
} by {
    if Real.0 <= x and x < Real.1 {
        floor_zero_of_unit_interval(x)
        floor(x) = Int.0
        frac(x) = x - Real.from_int(floor(x))
        (x - Real.from_int(Int.0) = x)
        (Real.from_int(Int.0) = Real.0)
        (x - Real.0 = x)
        frac(x) = x
    }
}

/// The circular distance of a number in [0, 1) is its distance from the nearer end.
theorem circular_distance_of_unit_interval(x: Real) {
    Real.0 <= x and x < Real.1 implies circular_distance(x) = x.min(Real.1 - x)
} by {
    if Real.0 <= x and x < Real.1 {
        frac_of_unit_interval(x)
        frac(x) = x
        circular_distance(x) = frac(x).min(Real.1 - frac(x))
        circular_distance(x) = x.min(Real.1 - x)
    }
}

/// If x lies in [b, 1 - b], its circular distance from the origin is at least b.
theorem cd_bound(x: Real, b: Real) {
    Real.0 <= x and x < Real.1 and b <= x and b <= Real.1 - x
    implies b <= circular_distance(x)
} by {
    if Real.0 <= x and x < Real.1 and b <= x and b <= Real.1 - x {
        circular_distance_of_unit_interval(x)
        circular_distance(x) = x.min(Real.1 - x)
        lte_min_of_bounds(b, x, Real.1 - x)
        b <= x.min(Real.1 - x)
        b <= circular_distance(x)
    }
}

// ---- the natural numbers embedded in the reals ----

/// A positive natural embeds as a positive real.
theorem from_nat_real_positive(n: Nat) {
    Nat.0 < n implies Real.0 < from_nat[Real](n)
} by {
    if Nat.0 < n {
        lt_imp_lte_suc(Nat.0, n)
        (Nat.0.suc <= n)
        (Nat.0.suc = Nat.1)
        Nat.1 <= n
        from_nat_real_lte(Nat.1, n)
        (from_nat[Real](Nat.1) <= from_nat[Real](n))
        (from_nat[Real](Nat.1) = Real.1)
        Real.1 <= from_nat[Real](n)
        lt_lte_trans(Real.0, Real.1, from_nat[Real](n))
        Real.0 < from_nat[Real](n)
    }
}

/// The embedding of the naturals into the reals preserves strict order.
theorem from_nat_real_lt(m: Nat, n: Nat) {
    m < n implies from_nat[Real](m) < from_nat[Real](n)
} by {
    if m < n {
        lt_imp_lte_suc(m, n)
        (m.suc <= n)
        (m.suc = m + Nat.1)
        (m + Nat.1 <= n)
        (from_nat[Real](m + Nat.1) <= from_nat[Real](n))
        (from_nat[Real](m + Nat.1) = from_nat[Real](m) + from_nat[Real](Nat.1))
        (from_nat[Real](Nat.1) = Real.1)
        (from_nat[Real](m) + Real.1 <= from_nat[Real](n))
        lt_add_one(from_nat[Real](m))
        (from_nat[Real](m) < from_nat[Real](m) + Real.1)
        lt_lte_trans(from_nat[Real](m), from_nat[Real](m) + Real.1, from_nat[Real](n))
        from_nat[Real](m) < from_nat[Real](n)
    }
}

/// A positive natural divides to one over itself.
theorem from_nat_div_self(n: Nat) {
    Nat.0 < n implies from_nat[Real](n) / from_nat[Real](n) = Real.1
} by {
    if Nat.0 < n {
        from_nat_real_positive(n)
        Real.0 < from_nat[Real](n)
        (from_nat[Real](n) != Real.0)
        mul_div_cancel(Real.1, from_nat[Real](n))
        (from_nat[Real](n) * (Real.1 / from_nat[Real](n)) = Real.1)
        mul_one_over(from_nat[Real](n), from_nat[Real](n))
        (from_nat[Real](n) * (Real.1 / from_nat[Real](n)) = from_nat[Real](n) / from_nat[Real](n))
        from_nat[Real](n) / from_nat[Real](n) = Real.1
    }
}

/// A proper fraction of naturals is below one.
theorem from_nat_frac_lt_one(k: Nat, n: Nat) {
    k < n implies from_nat[Real](k) / from_nat[Real](n) < Real.1
} by {
    if k < n {
        lt_imp_lte_suc(k, n)
        (k.suc <= n)
        lt_add_suc(Nat.0, k)
        (Nat.0 < Nat.0 + k.suc)
        (Nat.0 + k.suc = k.suc)
        Nat.0 < k.suc
        lt_and_lte(Nat.0, k.suc, n)
        Nat.0 < n
        from_nat_real_lt(k, n)
        (from_nat[Real](k) < from_nat[Real](n))
        from_nat_real_positive(n)
        (Real.0 < from_nat[Real](n))
        inverse_of_positive_is_positive(from_nat[Real](n))
        (Real.0 < from_nat[Real](n).inverse)
        mul_lt_mul_of_pos_right(from_nat[Real](k), from_nat[Real](n), from_nat[Real](n).inverse)
        (from_nat[Real](k) * from_nat[Real](n).inverse < from_nat[Real](n) * from_nat[Real](n).inverse)
        (from_nat[Real](n) != Real.0)
        mul_inverse(from_nat[Real](n))
        (from_nat[Real](n) * from_nat[Real](n).inverse = Real.1)
        (from_nat[Real](k) * from_nat[Real](n).inverse < Real.1)
        (from_nat[Real](k) * from_nat[Real](n).inverse = from_nat[Real](k) / from_nat[Real](n))
        from_nat[Real](k) / from_nat[Real](n) < Real.1
    }
}

/// Comparing fractions with the same positive denominator.
theorem same_denom_lte(a: Nat, b: Nat, d: Nat) {
    Nat.0 < d and a <= b
    implies from_nat[Real](a) / from_nat[Real](d) <= from_nat[Real](b) / from_nat[Real](d)
} by {
    if Nat.0 < d and a <= b {
        from_nat_real_lte(a, b)
        (from_nat[Real](a) <= from_nat[Real](b))
        from_nat_real_positive(d)
        (Real.0 < from_nat[Real](d))
        div_le_div_pos(from_nat[Real](a), from_nat[Real](b), from_nat[Real](d))
        from_nat[Real](a) / from_nat[Real](d) <= from_nat[Real](b) / from_nat[Real](d)
    }
}

/// Subtraction distributes over division.
theorem sub_div(a: Real, b: Real, c: Real) {
    c != Real.0 implies a / c - b / c = (a - b) / c
} by {
    if c != Real.0 {
        mul_sub_distrib_left(a, b, c.inverse)
        ((a - b) * c.inverse = a * c.inverse - b * c.inverse)
        ((a - b) / c = (a - b) * c.inverse)
        (a / c = a * c.inverse)
        (b / c = b * c.inverse)
        (a / c - b / c = a * c.inverse - b * c.inverse)
        a / c - b / c = (a - b) / c
    }
}

/// The complement of a unit fraction with denominator n is (n - k)/n.
theorem one_minus_frac(k: Nat, n: Nat) {
    k < n implies Real.1 - from_nat[Real](k) / from_nat[Real](n) = from_nat[Real](n - k) / from_nat[Real](n)
} by {
    if k < n {
        lt_imp_lte(k, n)
        k <= n
        lt_imp_lte_suc(k, n)
        (k.suc <= n)
        lt_add_suc(Nat.0, k)
        (Nat.0 < Nat.0 + k.suc)
        (Nat.0 + k.suc = k.suc)
        Nat.0 < k.suc
        lt_and_lte(Nat.0, k.suc, n)
        Nat.0 < n
        from_nat_div_self(n)
        (from_nat[Real](n) / from_nat[Real](n) = Real.1)
        (Real.1 - from_nat[Real](k) / from_nat[Real](n)
            = from_nat[Real](n) / from_nat[Real](n) - from_nat[Real](k) / from_nat[Real](n))
        from_nat_real_positive(n)
        (Real.0 < from_nat[Real](n))
        (from_nat[Real](n) != Real.0)
        sub_div(from_nat[Real](n), from_nat[Real](k), from_nat[Real](n))
        (from_nat[Real](n) / from_nat[Real](n) - from_nat[Real](k) / from_nat[Real](n)
            = (from_nat[Real](n) - from_nat[Real](k)) / from_nat[Real](n))
        add_sub(n, k)
        (n - k + k = n)
        add_comm(n - k, k)
        ((n - k) + k = k + (n - k))
        (k + (n - k) = n)
        (from_nat[Real](k + (n - k)) = from_nat[Real](k) + from_nat[Real](n - k))
        (from_nat[Real](n) = from_nat[Real](k) + from_nat[Real](n - k))
        sub_cancels(from_nat[Real](k), from_nat[Real](n - k))
        (from_nat[Real](k) + from_nat[Real](n - k) - from_nat[Real](k) = from_nat[Real](n - k))
        (from_nat[Real](n) - from_nat[Real](k) = from_nat[Real](n - k))
        (Real.1 - from_nat[Real](k) / from_nat[Real](n)
            = from_nat[Real](n - k) / from_nat[Real](n))
    }
}

/// The workhorse bound: at position k/n, the distance from the origin is at least 1/n.
theorem cd_bound_frac(k: Nat, n: Nat) {
    Nat.1 <= k and k < n implies
        Real.1 / from_nat[Real](n) <= circular_distance(from_nat[Real](k) / from_nat[Real](n))
} by {
    if Nat.1 <= k and k < n {
        lt_imp_lte_suc(k, n)
        (k.suc <= n)
        lt_add_suc(Nat.0, k)
        (Nat.0 < Nat.0 + k.suc)
        (Nat.0 + k.suc = k.suc)
        Nat.0 < k.suc
        lt_and_lte(Nat.0, k.suc, n)
        Nat.0 < n
        from_nat_real_nonnegative(k)
        (Real.0 <= from_nat[Real](k))
        from_nat_real_positive(n)
        (Real.0 < from_nat[Real](n))
        div_le_div_pos(Real.0, from_nat[Real](k), from_nat[Real](n))
        (Real.0 / from_nat[Real](n) <= from_nat[Real](k) / from_nat[Real](n))
        (Real.0 / from_nat[Real](n) = Real.0)
        (Real.0 <= from_nat[Real](k) / from_nat[Real](n))
        from_nat_frac_lt_one(k, n)
        (from_nat[Real](k) / from_nat[Real](n) < Real.1)
        same_denom_lte(Nat.1, k, n)
        (from_nat[Real](Nat.1) / from_nat[Real](n) <= from_nat[Real](k) / from_nat[Real](n))
        (from_nat[Real](Nat.1) = Real.1)
        (Real.1 / from_nat[Real](n) <= from_nat[Real](k) / from_nat[Real](n))
        one_minus_frac(k, n)
        (Real.1 - from_nat[Real](k) / from_nat[Real](n) = from_nat[Real](n - k) / from_nat[Real](n))
        sub_pos(n, k)
        (n - k > Nat.0)
        (Nat.0 < n - k)
        lt_imp_lte_suc(Nat.0, n - k)
        (Nat.1 <= n - k)
        same_denom_lte(Nat.1, n - k, n)
        (from_nat[Real](Nat.1) / from_nat[Real](n) <= from_nat[Real](n - k) / from_nat[Real](n))
        (from_nat[Real](Nat.1) = Real.1)
        (Real.1 / from_nat[Real](n) <= from_nat[Real](n - k) / from_nat[Real](n))
        (Real.1 / from_nat[Real](n) <= Real.1 - from_nat[Real](k) / from_nat[Real](n))
        cd_bound(from_nat[Real](k) / from_nat[Real](n), Real.1 / from_nat[Real](n))
        (Real.1 / from_nat[Real](n) <= circular_distance(from_nat[Real](k) / from_nat[Real](n)))
    }
}

// ---- concrete facts about the small natural numbers ----

theorem nat_zero_lt_two {
    Nat.0 < Nat.2
} by {
    lt_add_suc(Nat.0, Nat.1)
    Nat.0 < Nat.0 + Nat.1.suc
    (Nat.0 + Nat.1.suc = Nat.2)
    Nat.0 < Nat.2
}

theorem nat_zero_lt_three {
    Nat.0 < Nat.3
} by {
    lt_add_suc(Nat.0, Nat.2)
    Nat.0 < Nat.0 + Nat.2.suc
    (Nat.0 + Nat.2.suc = Nat.3)
    Nat.0 < Nat.3
}

theorem nat_zero_lt_four {
    Nat.0 < Nat.4
} by {
    lt_add_suc(Nat.0, Nat.3)
    Nat.0 < Nat.0 + Nat.3.suc
    (Nat.0 + Nat.3.suc = Nat.4)
    Nat.0 < Nat.4
}

theorem nat_zero_lt_five {
    Nat.0 < Nat.5
} by {
    lt_add_suc(Nat.0, Nat.4)
    Nat.0 < Nat.0 + Nat.4.suc
    (Nat.0 + Nat.4.suc = Nat.5)
    Nat.0 < Nat.5
}

theorem nat_one_le_one {
    Nat.1 <= Nat.1
} by {
    lte_ref(Nat.1)
}

theorem nat_lt_one_two {
    Nat.1 < Nat.2
} by {
    lt_add_suc(Nat.1, Nat.0)
    Nat.1 < Nat.1 + Nat.0.suc
    (Nat.1 + Nat.0.suc = Nat.2)
    Nat.1 < Nat.2
}

theorem nat_lt_one_three {
    Nat.1 < Nat.3
} by {
    lt_add_suc(Nat.1, Nat.1)
    Nat.1 < Nat.1 + Nat.1.suc
    (Nat.1 + Nat.1.suc = Nat.3)
    Nat.1 < Nat.3
}

theorem nat_lt_one_four {
    Nat.1 < Nat.4
} by {
    lt_add_suc(Nat.1, Nat.2)
    Nat.1 < Nat.1 + Nat.2.suc
    (Nat.1 + Nat.2.suc = Nat.4)
    Nat.1 < Nat.4
}

theorem nat_lt_two_four {
    Nat.2 < Nat.4
} by {
    lt_add_suc(Nat.2, Nat.1)
    Nat.2 < Nat.2 + Nat.1.suc
    (Nat.2 + Nat.1.suc = Nat.4)
    Nat.2 < Nat.4
}

theorem nat_lt_one_five {
    Nat.1 < Nat.5
} by {
    lt_add_suc(Nat.1, Nat.3)
    Nat.1 < Nat.1 + Nat.3.suc
    (Nat.1 + Nat.3.suc = Nat.5)
    Nat.1 < Nat.5
}

theorem nat_lt_two_five {
    Nat.2 < Nat.5
} by {
    lt_add_suc(Nat.2, Nat.2)
    Nat.2 < Nat.2 + Nat.2.suc
    (Nat.2 + Nat.2.suc = Nat.5)
    Nat.2 < Nat.5
}

theorem nat_lt_three_five {
    Nat.3 < Nat.5
} by {
    lt_add_suc(Nat.3, Nat.1)
    Nat.3 < Nat.3 + Nat.1.suc
    (Nat.1.suc = Nat.2)
    (Nat.3 + Nat.1.suc = Nat.3 + Nat.2)
    (Nat.3 + Nat.2 = Nat.5)
    Nat.3 < Nat.5
}

theorem nat_one_le_two {
    Nat.1 <= Nat.2
} by {
    nat_lt_one_two
    (Nat.1 < Nat.2)
    lt_imp_lte(Nat.1, Nat.2)
    Nat.1 <= Nat.2
}

theorem nat_one_le_three {
    Nat.1 <= Nat.3
} by {
    nat_lt_one_three
    (Nat.1 < Nat.3)
    lt_imp_lte(Nat.1, Nat.3)
    Nat.1 <= Nat.3
}

theorem nat_one_le_four {
    Nat.1 <= Nat.4
} by {
    nat_lt_one_four
    (Nat.1 < Nat.4)
    lt_imp_lte(Nat.1, Nat.4)
    Nat.1 <= Nat.4
}

// ---- the small cases ----

/// One runner: nobody moves, so the lonely condition is vacuous.
theorem lonely_runner_one {
    exists(t: Real) {
        forall(i: Nat) {
            Nat.1 <= i and i < Nat.1 implies
                Real.1 / from_nat[Real](Nat.2) <= circular_distance(from_nat[Real](Nat.0) * t)
        }
    }
} by {
    exists_intro(function(t: Real) {
        forall(i: Nat) {
            Nat.1 <= i and i < Nat.1 implies
                Real.1 / from_nat[Real](Nat.2) <= circular_distance(from_nat[Real](Nat.0) * t)
        }
    }, Real.0)
    forall(i: Nat) {
        if Nat.1 <= i and i < Nat.1 {
            lte_and_lt(Nat.1, i, Nat.1)
            (Nat.1 <= i and i < Nat.1 implies Nat.1 < Nat.1)
            Nat.1 < Nat.1
            lt_not_ref(Nat.1)
            false
        }
    }
}

/// Two runners (one moving): at t = 1/(3v) the moving runner is at 1/3, exactly at
/// the required distance 1/3 from the origin.
theorem lonely_runner_two(v: Real) {
    Real.1 <= v implies exists(t: Real) {
        Real.1 / from_nat[Real](Nat.3) <= circular_distance(v * t)
    }
} by {
    if Real.1 <= v {
        lt_lte_trans(Real.0, Real.1, v)
        Real.0 < v
        lt_imp_ne(Real.0, v)
        (Real.0 != v)
        (v != Real.0)
        from_nat_real_positive(Nat.3)
        (Real.0 < from_nat[Real](Nat.3))
        lt_imp_ne(Real.0, from_nat[Real](Nat.3))
        (from_nat[Real](Nat.3) != Real.0)
        mul_not_zero(from_nat[Real](Nat.3), v)
        (from_nat[Real](Nat.3) * v != Real.0)
        mul_one_over(v, from_nat[Real](Nat.3) * v)
        (v * (Real.1 / (from_nat[Real](Nat.3) * v)) = v / (from_nat[Real](Nat.3) * v))
        mul_one_left(v)
        (Real.1 * v = v)
        (v / (from_nat[Real](Nat.3) * v) = (Real.1 * v) / (from_nat[Real](Nat.3) * v))
        div_cancel_common(Real.1, v, from_nat[Real](Nat.3))
        ((Real.1 * v) / (from_nat[Real](Nat.3) * v) = Real.1 / from_nat[Real](Nat.3))
        (v * (Real.1 / (from_nat[Real](Nat.3) * v)) = Real.1 / from_nat[Real](Nat.3))
        cd_bound_frac(Nat.1, Nat.3)
        (Nat.1 <= Nat.1 and Nat.1 < Nat.3)
        (Real.1 / from_nat[Real](Nat.3) <= circular_distance(from_nat[Real](Nat.1) / from_nat[Real](Nat.3)))
        (from_nat[Real](Nat.1) = Real.1)
        (Real.1 / from_nat[Real](Nat.3) <= circular_distance(Real.1 / from_nat[Real](Nat.3)))
        (Real.1 / from_nat[Real](Nat.3) <= circular_distance(v * (Real.1 / (from_nat[Real](Nat.3) * v))))
        exists_intro(function(t: Real) {
            Real.1 / from_nat[Real](Nat.3) <= circular_distance(v * t)
        }, Real.1 / (from_nat[Real](Nat.3) * v))
        (Real.1 / from_nat[Real](Nat.3) <= circular_distance(v * (Real.1 / (from_nat[Real](Nat.3) * v))))
        exists(t: Real) {
            Real.1 / from_nat[Real](Nat.3) <= circular_distance(v * t)
        }
    }
}

/// Three runners (two moving, speeds 1 and 2): at t = 1/4 the runners are at 1/4
/// and 1/2, both at distance at least 1/4 from the origin.
theorem lonely_runner_three {
    exists(t: Real) {
        Real.1 / from_nat[Real](Nat.4) <= circular_distance(Real.1 * t)
            and Real.1 / from_nat[Real](Nat.4) <= circular_distance(from_nat[Real](Nat.2) * t)
    }
} by {
    mul_one_left(Real.1 / from_nat[Real](Nat.4))
    (Real.1 * (Real.1 / from_nat[Real](Nat.4)) = Real.1 / from_nat[Real](Nat.4))
    cd_bound_frac(Nat.1, Nat.4)
    (Nat.1 <= Nat.1 and Nat.1 < Nat.4)
    (Real.1 / from_nat[Real](Nat.4) <= circular_distance(from_nat[Real](Nat.1) / from_nat[Real](Nat.4)))
    (from_nat[Real](Nat.1) = Real.1)
    (Real.1 / from_nat[Real](Nat.4) <= circular_distance(Real.1 / from_nat[Real](Nat.4)))
    (Real.1 / from_nat[Real](Nat.4) <= circular_distance(Real.1 * (Real.1 / from_nat[Real](Nat.4))))
    from_nat_real_positive(Nat.4)
    (Real.0 < from_nat[Real](Nat.4))
    (from_nat[Real](Nat.4) != Real.0)
    mul_one_over(from_nat[Real](Nat.2), from_nat[Real](Nat.4))
    (from_nat[Real](Nat.2) * (Real.1 / from_nat[Real](Nat.4)) = from_nat[Real](Nat.2) / from_nat[Real](Nat.4))
    cd_bound_frac(Nat.2, Nat.4)
    (Nat.1 <= Nat.2 and Nat.2 < Nat.4)
    (Real.1 / from_nat[Real](Nat.4) <= circular_distance(from_nat[Real](Nat.2) / from_nat[Real](Nat.4)))
    (Real.1 / from_nat[Real](Nat.4) <= circular_distance(from_nat[Real](Nat.2) * (Real.1 / from_nat[Real](Nat.4))))
    (Real.1 / from_nat[Real](Nat.4)
        <= circular_distance(Real.1 * (Real.1 / from_nat[Real](Nat.4)))
        and Real.1 / from_nat[Real](Nat.4)
            <= circular_distance(from_nat[Real](Nat.2) * (Real.1 / from_nat[Real](Nat.4))))
    exists(t: Real) {
        Real.1 / from_nat[Real](Nat.4) <= circular_distance(Real.1 * t)
            and Real.1 / from_nat[Real](Nat.4) <= circular_distance(from_nat[Real](Nat.2) * t)
    }
}

/// Four runners (three moving, speeds 1, 2 and 3): at t = 1/5 the runners are at
/// 1/5, 2/5 and 3/5, all at distance at least 1/5 from the origin.
theorem lonely_runner_four {
    exists(t: Real) {
        Real.1 / from_nat[Real](Nat.5) <= circular_distance(Real.1 * t)
            and Real.1 / from_nat[Real](Nat.5) <= circular_distance(from_nat[Real](Nat.2) * t)
            and Real.1 / from_nat[Real](Nat.5) <= circular_distance(from_nat[Real](Nat.3) * t)
    }
} by {
    mul_one_left(Real.1 / from_nat[Real](Nat.5))
    (Real.1 * (Real.1 / from_nat[Real](Nat.5)) = Real.1 / from_nat[Real](Nat.5))
    cd_bound_frac(Nat.1, Nat.5)
    (Nat.1 <= Nat.1 and Nat.1 < Nat.5)
    (Real.1 / from_nat[Real](Nat.5) <= circular_distance(from_nat[Real](Nat.1) / from_nat[Real](Nat.5)))
    (from_nat[Real](Nat.1) = Real.1)
    (Real.1 / from_nat[Real](Nat.5) <= circular_distance(Real.1 / from_nat[Real](Nat.5)))
    (Real.1 / from_nat[Real](Nat.5) <= circular_distance(Real.1 * (Real.1 / from_nat[Real](Nat.5))))
    from_nat_real_positive(Nat.5)
    (Real.0 < from_nat[Real](Nat.5))
    (from_nat[Real](Nat.5) != Real.0)
    mul_one_over(from_nat[Real](Nat.2), from_nat[Real](Nat.5))
    (from_nat[Real](Nat.2) * (Real.1 / from_nat[Real](Nat.5)) = from_nat[Real](Nat.2) / from_nat[Real](Nat.5))
    cd_bound_frac(Nat.2, Nat.5)
    (Nat.1 <= Nat.2 and Nat.2 < Nat.5)
    (Real.1 / from_nat[Real](Nat.5) <= circular_distance(from_nat[Real](Nat.2) / from_nat[Real](Nat.5)))
    (Real.1 / from_nat[Real](Nat.5) <= circular_distance(from_nat[Real](Nat.2) * (Real.1 / from_nat[Real](Nat.5))))
    mul_one_over(from_nat[Real](Nat.3), from_nat[Real](Nat.5))
    (from_nat[Real](Nat.3) * (Real.1 / from_nat[Real](Nat.5)) = from_nat[Real](Nat.3) / from_nat[Real](Nat.5))
    cd_bound_frac(Nat.3, Nat.5)
    (Nat.1 <= Nat.3 and Nat.3 < Nat.5)
    (Real.1 / from_nat[Real](Nat.5) <= circular_distance(from_nat[Real](Nat.3) / from_nat[Real](Nat.5)))
    (Real.1 / from_nat[Real](Nat.5) <= circular_distance(from_nat[Real](Nat.3) * (Real.1 / from_nat[Real](Nat.5))))
    (Real.1 / from_nat[Real](Nat.5)
        <= circular_distance(Real.1 * (Real.1 / from_nat[Real](Nat.5)))
        and Real.1 / from_nat[Real](Nat.5)
            <= circular_distance(from_nat[Real](Nat.2) * (Real.1 / from_nat[Real](Nat.5)))
        and Real.1 / from_nat[Real](Nat.5)
            <= circular_distance(from_nat[Real](Nat.3) * (Real.1 / from_nat[Real](Nat.5))))
    exists(t: Real) {
        Real.1 / from_nat[Real](Nat.5) <= circular_distance(Real.1 * t)
            and Real.1 / from_nat[Real](Nat.5) <= circular_distance(from_nat[Real](Nat.2) * t)
            and Real.1 / from_nat[Real](Nat.5) <= circular_distance(from_nat[Real](Nat.3) * t)
    }
}

// ---- the general conjecture ----
//
// The lonely runner conjecture states that for any number n of runners, with runner 0
// stationary and runners 1..n-1 moving at distinct positive integer speeds, there is a
// time at which every moving runner is at circular distance at least 1/(n+1) from the
// origin. It is an open problem in general (known for up to seven moving runners), so
// the theorem below is stated but not proved here.
//
// theorem lonely_runner_conjecture(n: Nat, v: Nat -> Nat) {
//     v(Nat.0) = Nat.0
//         and forall(i: Nat) {
//             Nat.1 <= i and i < n implies v(i) < v(i + Nat.1)
//         }
//     implies exists(t: Real) {
//         forall(i: Nat) {
//             Nat.1 <= i and i < n implies
//                 Real.1 / from_nat[Real](n + Nat.1) <= circular_distance(from_nat[Real](v(i)) * t)
//         }
//     }
// } by {
//     // Open problem; not proved in this file.
// }
