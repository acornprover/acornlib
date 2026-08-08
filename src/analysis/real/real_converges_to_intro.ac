from nat import Nat
from real import Real, converges_to, tail_bound

/// The epsilon condition gives convergence.
///
/// `src/real/` supplies `converges_to` and reads tail bounds out of it, but nothing builds one
/// from the pointwise condition. Any argument that transports convergence along a construction
/// has to, so the unfolding is stated once here rather than at each such site.
theorem converges_to_intro(q: Nat -> Real, a: Real) {
    (forall(eps: Real) {
        eps.is_positive implies exists(n: Nat) { tail_bound(q, a, n, eps) }
    }) implies converges_to(q, a)
} by {
    (converges_to(q, a) = forall(eps: Real) {
        eps.is_positive implies exists(n: Nat) { tail_bound(q, a, n, eps) }
    })
}
