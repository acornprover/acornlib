from nat import Nat
from real import Real, lte_trans, add_seq, converges, converges_to, limit, seq_lte,
    seq_lte_preserves_limit, limit_add_seq, converges_to_imp_converges,
    converges_to_unique, convergent_converges_to_limit
from analysis.real.real_tail_supremum import is_bounded_above
from analysis.real.real_limsup import is_bounded_below, neg_tail_sup, neg_tail_sup_converges
from analysis.real.real_liminf import liminf, tail_inf, tail_inf_bounds_terms, tail_inf_is_greatest,
    neg_seq, neg_seq_bounded_above, neg_seq_bounded_below, liminf_eq_limit_tail_inf,
    neg_tail_sup_neg_seq_eq_tail_inf
from analysis.real.real_limsup_add import add_seq_bounded_above, add_seq_bounded_below

/// The tail infimum of a sum is at least the sum of the tail infima.
///
/// At the level of the tails, where it is one application of each characterising property.
theorem tail_inf_add(a: Nat -> Real, b: Nat -> Real, n: Nat) {
    is_bounded_below(a) and is_bounded_below(b)
        implies tail_inf(a, n) + tail_inf(b, n) <= tail_inf(add_seq(a, b), n)
} by {
    if is_bounded_below(a) and is_bounded_below(b) {
        add_seq_bounded_below(a, b)
        is_bounded_below(add_seq(a, b))
        forall(k: Nat) {
            if n <= k {
                tail_inf_bounds_terms(a, n, k)
                tail_inf(a, n) <= a(k)
                tail_inf_bounds_terms(b, n, k)
                tail_inf(b, n) <= b(k)
                (tail_inf(a, n) + tail_inf(b, n) <= a(k) + b(k))
                (add_seq(a, b)(k) = a(k) + b(k))
                tail_inf(a, n) + tail_inf(b, n) <= add_seq(a, b)(k)
            }
            (n <= k implies tail_inf(a, n) + tail_inf(b, n) <= add_seq(a, b)(k))
        }
        tail_inf_is_greatest(add_seq(a, b), n, tail_inf(a, n) + tail_inf(b, n))
        tail_inf(a, n) + tail_inf(b, n) <= tail_inf(add_seq(a, b), n)
    }
}

/// The limit inferior of a sum is at least the sum of the limit inferiors.
theorem liminf_add(a: Nat -> Real, b: Nat -> Real) {
    is_bounded_above(a) and is_bounded_below(a) and is_bounded_above(b)
        and is_bounded_below(b)
        implies liminf(a) + liminf(b) <= liminf(add_seq(a, b))
} by {
    if is_bounded_above(a) and is_bounded_below(a) and is_bounded_above(b)
        and is_bounded_below(b) {
        add_seq_bounded_above(a, b)
        is_bounded_above(add_seq(a, b))
        add_seq_bounded_below(a, b)
        is_bounded_below(add_seq(a, b))
        neg_seq_bounded_above(a)
        is_bounded_above(neg_seq(a))
        neg_seq_bounded_below(a)
        is_bounded_below(neg_seq(a))
        neg_seq_bounded_above(b)
        is_bounded_above(neg_seq(b))
        neg_seq_bounded_below(b)
        is_bounded_below(neg_seq(b))
        neg_seq_bounded_above(add_seq(a, b))
        is_bounded_above(neg_seq(add_seq(a, b)))
        neg_seq_bounded_below(add_seq(a, b))
        is_bounded_below(neg_seq(add_seq(a, b)))
        neg_tail_sup_converges(neg_seq(a))
        converges(neg_tail_sup(neg_seq(a)))
        neg_tail_sup_converges(neg_seq(b))
        converges(neg_tail_sup(neg_seq(b)))
        neg_tail_sup_converges(neg_seq(add_seq(a, b)))
        converges(neg_tail_sup(neg_seq(add_seq(a, b))))
        limit_add_seq(neg_tail_sup(neg_seq(a)), neg_tail_sup(neg_seq(b)))
        converges_to(add_seq(neg_tail_sup(neg_seq(a)), neg_tail_sup(neg_seq(b))),
            limit(neg_tail_sup(neg_seq(a))) + limit(neg_tail_sup(neg_seq(b))))
        converges_to_imp_converges(
            add_seq(neg_tail_sup(neg_seq(a)), neg_tail_sup(neg_seq(b))),
            limit(neg_tail_sup(neg_seq(a))) + limit(neg_tail_sup(neg_seq(b))))
        converges(add_seq(neg_tail_sup(neg_seq(a)), neg_tail_sup(neg_seq(b))))
        convergent_converges_to_limit(
            add_seq(neg_tail_sup(neg_seq(a)), neg_tail_sup(neg_seq(b))))
        converges_to(add_seq(neg_tail_sup(neg_seq(a)), neg_tail_sup(neg_seq(b))),
            limit(add_seq(neg_tail_sup(neg_seq(a)), neg_tail_sup(neg_seq(b)))))
        converges_to_unique(
            add_seq(neg_tail_sup(neg_seq(a)), neg_tail_sup(neg_seq(b))),
            limit(neg_tail_sup(neg_seq(a))) + limit(neg_tail_sup(neg_seq(b))),
            limit(add_seq(neg_tail_sup(neg_seq(a)), neg_tail_sup(neg_seq(b)))))
        (limit(neg_tail_sup(neg_seq(a))) + limit(neg_tail_sup(neg_seq(b)))
            = limit(add_seq(neg_tail_sup(neg_seq(a)), neg_tail_sup(neg_seq(b)))))
        forall(n: Nat) {
            tail_inf_add(a, b, n)
            tail_inf(a, n) + tail_inf(b, n) <= tail_inf(add_seq(a, b), n)
            neg_tail_sup_neg_seq_eq_tail_inf(a, n)
            neg_tail_sup(neg_seq(a))(n) = tail_inf(a, n)
            neg_tail_sup_neg_seq_eq_tail_inf(b, n)
            neg_tail_sup(neg_seq(b))(n) = tail_inf(b, n)
            neg_tail_sup_neg_seq_eq_tail_inf(add_seq(a, b), n)
            neg_tail_sup(neg_seq(add_seq(a, b)))(n) = tail_inf(add_seq(a, b), n)
            (add_seq(neg_tail_sup(neg_seq(a)), neg_tail_sup(neg_seq(b)))(n)
                = neg_tail_sup(neg_seq(a))(n) + neg_tail_sup(neg_seq(b))(n))
            (add_seq(neg_tail_sup(neg_seq(a)), neg_tail_sup(neg_seq(b)))(n)
                <= neg_tail_sup(neg_seq(add_seq(a, b)))(n))
        }
        (seq_lte(add_seq(neg_tail_sup(neg_seq(a)), neg_tail_sup(neg_seq(b))),
            neg_tail_sup(neg_seq(add_seq(a, b)))) = forall(n: Nat) {
            add_seq(neg_tail_sup(neg_seq(a)), neg_tail_sup(neg_seq(b)))(n)
                <= neg_tail_sup(neg_seq(add_seq(a, b)))(n)
        })
        seq_lte(add_seq(neg_tail_sup(neg_seq(a)), neg_tail_sup(neg_seq(b))),
            neg_tail_sup(neg_seq(add_seq(a, b))))
        seq_lte_preserves_limit(
            add_seq(neg_tail_sup(neg_seq(a)), neg_tail_sup(neg_seq(b))),
            neg_tail_sup(neg_seq(add_seq(a, b))))
        (limit(add_seq(neg_tail_sup(neg_seq(a)), neg_tail_sup(neg_seq(b))))
            <= limit(neg_tail_sup(neg_seq(add_seq(a, b)))))
        liminf_eq_limit_tail_inf(a)
        liminf(a) = limit(neg_tail_sup(neg_seq(a)))
        liminf_eq_limit_tail_inf(b)
        liminf(b) = limit(neg_tail_sup(neg_seq(b)))
        liminf_eq_limit_tail_inf(add_seq(a, b))
        liminf(add_seq(a, b)) = limit(neg_tail_sup(neg_seq(add_seq(a, b))))
        liminf(a) + liminf(b) <= liminf(add_seq(a, b))
    }
}
