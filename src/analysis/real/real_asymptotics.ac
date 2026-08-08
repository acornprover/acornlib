/// Eventual comparison and asymptotic notation for real-valued sequences.
///
/// This module is deliberately a consumer of the accepted generic Nat-eventual
/// API (`nat_eventually`). It does not define a second generic eventual predicate
/// and it does not edit or depend on the public `real` interface barrel beyond
/// the already exported real primitives used here.

from nat import Nat
from order import lte_trans, max_imp_gte
from data.nat.nat_eventually import eventually_after_nat, eventually_after_nat_at,
    eventually_predicate_nat, eventually_predicate_nat_has_witness,
    eventually_predicate_nat_intro, eventually_predicate_nat_of_forall
from real import Real, abs_gte_zero, mul_nonneg

/// Eventual pointwise order between real sequences.
define eventually_seq_lte(a: Nat -> Real, b: Nat -> Real) -> Bool {
    eventually_predicate_nat(function(n: Nat) { a(n) <= b(n) })
}

/// Eventual equality between real sequences.
define eventually_seq_eq(a: Nat -> Real, b: Nat -> Real) -> Bool {
    eventually_predicate_nat(function(n: Nat) { a(n) = b(n) })
}

/// Big-O comparison for real sequences: `|a n|` is eventually bounded by `C |b n|`.
define is_big_o_seq(a: Nat -> Real, b: Nat -> Real) -> Bool {
    exists(c: Real) {
        c.is_positive and eventually_predicate_nat(function(n: Nat) {
            a(n).abs <= c * b(n).abs
        })
    }
}

/// Little-o comparison for real sequences: every positive epsilon eventually
/// bounds `|a n|` by `eps |b n|`.
define is_little_o_seq(a: Nat -> Real, b: Nat -> Real) -> Bool {
    forall(eps: Real) {
        eps.is_positive implies eventually_predicate_nat(function(n: Nat) {
            a(n).abs <= eps * b(n).abs
        })
    }
}

/// Eventual equality is reflexive.
theorem eventually_seq_eq_refl(a: Nat -> Real) {
    eventually_seq_eq(a, a)
} by {
    forall(i: Nat) {
        a(i) = a(i)
    }
    eventually_predicate_nat_of_forall(function(i: Nat) { a(i) = a(i) })
    eventually_predicate_nat(function(i: Nat) { a(i) = a(i) })
    eventually_seq_eq(a, a) = eventually_predicate_nat(function(i: Nat) { a(i) = a(i) })
    eventually_seq_eq(a, a)
}

/// Eventual equality is symmetric.
theorem eventually_seq_eq_symm(a: Nat -> Real, b: Nat -> Real) {
    eventually_seq_eq(a, b) implies eventually_seq_eq(b, a)
} by {
    if eventually_seq_eq(a, b) {
        eventually_seq_eq(a, b) = eventually_predicate_nat(function(i: Nat) { a(i) = b(i) })
        eventually_predicate_nat(function(i: Nat) { a(i) = b(i) })
        eventually_predicate_nat_has_witness(function(i: Nat) { a(i) = b(i) })
        let n0: Nat satisfy {
            eventually_after_nat(function(i: Nat) { a(i) = b(i) }, n0)
        }
        forall(i: Nat) {
            if n0 <= i {
                eventually_after_nat_at(function(k: Nat) { a(k) = b(k) }, n0, i)
                a(i) = b(i)
                b(i) = a(i)
                function(k: Nat) { b(k) = a(k) }(i)
            }
        }
        eventually_after_nat(function(k: Nat) { b(k) = a(k) }, n0)
        eventually_predicate_nat_intro(function(k: Nat) { b(k) = a(k) }, n0)
        eventually_predicate_nat(function(k: Nat) { b(k) = a(k) })
        eventually_seq_eq(b, a) = eventually_predicate_nat(function(i: Nat) { b(i) = a(i) })
        eventually_seq_eq(b, a)
    }
}

/// Eventual equality is transitive.
theorem eventually_seq_eq_trans(a: Nat -> Real, b: Nat -> Real, c: Nat -> Real) {
    eventually_seq_eq(a, b) and eventually_seq_eq(b, c) implies eventually_seq_eq(a, c)
} by {
    if eventually_seq_eq(a, b) and eventually_seq_eq(b, c) {
        eventually_seq_eq(a, b) = eventually_predicate_nat(function(i: Nat) { a(i) = b(i) })
        eventually_seq_eq(b, c) = eventually_predicate_nat(function(i: Nat) { b(i) = c(i) })
        eventually_predicate_nat(function(i: Nat) { a(i) = b(i) })
        eventually_predicate_nat(function(i: Nat) { b(i) = c(i) })
        eventually_predicate_nat_has_witness(function(i: Nat) { a(i) = b(i) })
        eventually_predicate_nat_has_witness(function(i: Nat) { b(i) = c(i) })
        let n1: Nat satisfy {
            eventually_after_nat(function(i: Nat) { a(i) = b(i) }, n1)
        }
        let n2: Nat satisfy {
            eventually_after_nat(function(i: Nat) { b(i) = c(i) }, n2)
        }
        let n0 = n1.max(n2)
        max_imp_gte[Nat](n1, n2)
        n1.max(n2) >= n1 and n1.max(n2) >= n2
        n1 <= n0
        n2 <= n0
        forall(i: Nat) {
            if n0 <= i {
                lte_trans(n1, n0, i)
                lte_trans(n2, n0, i)
                eventually_after_nat_at(function(k: Nat) { a(k) = b(k) }, n1, i)
                eventually_after_nat_at(function(k: Nat) { b(k) = c(k) }, n2, i)
                a(i) = b(i)
                b(i) = c(i)
                a(i) = c(i)
                function(k: Nat) { a(k) = c(k) }(i)
            }
        }
        eventually_after_nat(function(k: Nat) { a(k) = c(k) }, n0)
        eventually_predicate_nat_intro(function(k: Nat) { a(k) = c(k) }, n0)
        eventually_predicate_nat(function(k: Nat) { a(k) = c(k) })
        eventually_seq_eq(a, c) = eventually_predicate_nat(function(k: Nat) { a(k) = c(k) })
        eventually_seq_eq(a, c)
    }
}

/// Eventual sequence order is reflexive.
theorem eventually_seq_lte_refl(a: Nat -> Real) {
    eventually_seq_lte(a, a)
} by {
    forall(i: Nat) {
        a(i) <= a(i)
    }
    eventually_predicate_nat_of_forall(function(i: Nat) { a(i) <= a(i) })
    eventually_predicate_nat(function(i: Nat) { a(i) <= a(i) })
    eventually_seq_lte(a, a) = eventually_predicate_nat(function(i: Nat) { a(i) <= a(i) })
    eventually_seq_lte(a, a)
}

/// Eventual sequence order is transitive.
theorem eventually_seq_lte_trans(a: Nat -> Real, b: Nat -> Real, c: Nat -> Real) {
    eventually_seq_lte(a, b) and eventually_seq_lte(b, c) implies eventually_seq_lte(a, c)
} by {
    if eventually_seq_lte(a, b) and eventually_seq_lte(b, c) {
        eventually_seq_lte(a, b) = eventually_predicate_nat(function(i: Nat) { a(i) <= b(i) })
        eventually_seq_lte(b, c) = eventually_predicate_nat(function(i: Nat) { b(i) <= c(i) })
        eventually_predicate_nat(function(i: Nat) { a(i) <= b(i) })
        eventually_predicate_nat(function(i: Nat) { b(i) <= c(i) })
        eventually_predicate_nat_has_witness(function(i: Nat) { a(i) <= b(i) })
        eventually_predicate_nat_has_witness(function(i: Nat) { b(i) <= c(i) })
        let n1: Nat satisfy {
            eventually_after_nat(function(i: Nat) { a(i) <= b(i) }, n1)
        }
        let n2: Nat satisfy {
            eventually_after_nat(function(i: Nat) { b(i) <= c(i) }, n2)
        }
        let n0 = n1.max(n2)
        max_imp_gte[Nat](n1, n2)
        n1.max(n2) >= n1 and n1.max(n2) >= n2
        n1 <= n0
        n2 <= n0
        forall(i: Nat) {
            if n0 <= i {
                lte_trans(n1, n0, i)
                lte_trans(n2, n0, i)
                eventually_after_nat_at(function(k: Nat) { a(k) <= b(k) }, n1, i)
                eventually_after_nat_at(function(k: Nat) { b(k) <= c(k) }, n2, i)
                a(i) <= b(i)
                b(i) <= c(i)
                lte_trans(a(i), b(i), c(i))
                a(i) <= c(i)
                function(k: Nat) { a(k) <= c(k) }(i)
            }
        }
        eventually_after_nat(function(k: Nat) { a(k) <= c(k) }, n0)
        eventually_predicate_nat_intro(function(k: Nat) { a(k) <= c(k) }, n0)
        eventually_predicate_nat(function(k: Nat) { a(k) <= c(k) })
        eventually_seq_lte(a, c) = eventually_predicate_nat(function(k: Nat) { a(k) <= c(k) })
        eventually_seq_lte(a, c)
    }
}

/// Introduce big-O from an explicit positive constant and eventual absolute-value bound.
theorem big_o_seq_intro(a: Nat -> Real, b: Nat -> Real, c: Real) {
    c.is_positive and eventually_predicate_nat(function(n: Nat) { a(n).abs <= c * b(n).abs })
    implies is_big_o_seq(a, b)
} by {
    if c.is_positive and eventually_predicate_nat(function(n: Nat) { a(n).abs <= c * b(n).abs }) {
        is_big_o_seq(a, b)
    }
}

/// An eventual bound by `|b|` gives a big-O bound with constant `1`.
theorem big_o_seq_of_eventual_abs_lte(a: Nat -> Real, b: Nat -> Real) {
    eventually_predicate_nat(function(n: Nat) { a(n).abs <= b(n).abs }) implies is_big_o_seq(a, b)
} by {
    if eventually_predicate_nat(function(n: Nat) { a(n).abs <= b(n).abs }) {
        Real.1.is_positive
        forall(n: Nat) {
            if a(n).abs <= b(n).abs {
                Real.1 * b(n).abs = b(n).abs
                a(n).abs <= Real.1 * b(n).abs
                function(k: Nat) { a(k).abs <= Real.1 * b(k).abs }(n)
            }
        }
        eventually_predicate_nat_has_witness(function(n: Nat) { a(n).abs <= b(n).abs })
        let n0: Nat satisfy {
            eventually_after_nat(function(n: Nat) { a(n).abs <= b(n).abs }, n0)
        }
        forall(n: Nat) {
            if n0 <= n {
                eventually_after_nat_at(function(k: Nat) { a(k).abs <= b(k).abs }, n0, n)
                a(n).abs <= b(n).abs
                Real.1 * b(n).abs = b(n).abs
                a(n).abs <= Real.1 * b(n).abs
                function(k: Nat) { a(k).abs <= Real.1 * b(k).abs }(n)
            }
        }
        eventually_after_nat(function(k: Nat) { a(k).abs <= Real.1 * b(k).abs }, n0)
        eventually_predicate_nat_intro(function(k: Nat) { a(k).abs <= Real.1 * b(k).abs }, n0)
        eventually_predicate_nat(function(k: Nat) { a(k).abs <= Real.1 * b(k).abs })
        big_o_seq_intro(a, b, Real.1)
        is_big_o_seq(a, b)
    }
}

/// Big-O is reflexive.
theorem big_o_seq_refl(a: Nat -> Real) {
    is_big_o_seq(a, a)
} by {
    forall(n: Nat) {
        a(n).abs <= a(n).abs
    }
    eventually_predicate_nat_of_forall(function(n: Nat) { a(n).abs <= a(n).abs })
    big_o_seq_of_eventual_abs_lte(a, a)
}

/// The zero sequence is big-O of every real sequence.
theorem big_o_seq_zero_left(b: Nat -> Real) {
    is_big_o_seq(function(n: Nat) { Real.0 }, b)
} by {
    Real.1.is_positive
    forall(n: Nat) {
        function(k: Nat) { Real.0 }(n).abs = Real.0
        abs_gte_zero(b(n))
        b(n).abs >= Real.0
        Real.1 * b(n).abs = b(n).abs
        function(k: Nat) { Real.0 }(n).abs <= b(n).abs
        function(k: Nat) { Real.0 }(n).abs <= Real.1 * b(n).abs
    }
    eventually_predicate_nat_of_forall(
        function(n: Nat) { function(k: Nat) { Real.0 }(n).abs <= Real.1 * b(n).abs })
    eventually_predicate_nat(function(n: Nat) { function(k: Nat) { Real.0 }(n).abs <= Real.1 * b(n).abs })
    big_o_seq_intro(function(k: Nat) { Real.0 }, b, Real.1)
    is_big_o_seq(function(k: Nat) { Real.0 }, b)
}

/// The zero sequence is little-o of every real sequence.
theorem little_o_seq_zero_left(b: Nat -> Real) {
    is_little_o_seq(function(n: Nat) { Real.0 }, b)
} by {
    forall(eps: Real) {
        if eps.is_positive {
            forall(n: Nat) {
                function(k: Nat) { Real.0 }(n).abs = Real.0
                eps >= Real.0
                abs_gte_zero(b(n))
                b(n).abs >= Real.0
                mul_nonneg(eps, b(n).abs)
                eps * b(n).abs >= Real.0
                function(k: Nat) { Real.0 }(n).abs <= eps * b(n).abs
            }
            eventually_predicate_nat_of_forall(
                function(n: Nat) { function(k: Nat) { Real.0 }(n).abs <= eps * b(n).abs })
            eventually_predicate_nat(function(n: Nat) { function(k: Nat) { Real.0 }(n).abs <= eps * b(n).abs })
        }
    }
    is_little_o_seq(function(k: Nat) { Real.0 }, b) = forall(eps: Real) {
        eps.is_positive implies eventually_predicate_nat(function(n: Nat) {
            function(k: Nat) { Real.0 }(n).abs <= eps * b(n).abs
        })
    }
    is_little_o_seq(function(k: Nat) { Real.0 }, b)
}

/// Little-o implies big-O, using epsilon `1`.
theorem little_o_seq_imp_big_o_seq(a: Nat -> Real, b: Nat -> Real) {
    is_little_o_seq(a, b) implies is_big_o_seq(a, b)
} by {
    if is_little_o_seq(a, b) {
        Real.1.is_positive
        is_little_o_seq(a, b) = forall(eps: Real) {
            eps.is_positive implies eventually_predicate_nat(function(n: Nat) {
                a(n).abs <= eps * b(n).abs
            })
        }
        eventually_predicate_nat(function(n: Nat) { a(n).abs <= Real.1 * b(n).abs })
        big_o_seq_intro(a, b, Real.1)
        is_big_o_seq(a, b)
    }
}

/// Big-O is preserved by eventual equality on the left sequence.
theorem big_o_seq_congr_left(a: Nat -> Real, a2: Nat -> Real, b: Nat -> Real) {
    eventually_seq_eq(a, a2) and is_big_o_seq(a, b) implies is_big_o_seq(a2, b)
} by {
    if eventually_seq_eq(a, a2) and is_big_o_seq(a, b) {
        let c: Real satisfy {
            c.is_positive and eventually_predicate_nat(function(n: Nat) { a(n).abs <= c * b(n).abs })
        }
        eventually_seq_eq(a, a2) = eventually_predicate_nat(function(n: Nat) { a(n) = a2(n) })
        eventually_predicate_nat(function(n: Nat) { a(n) = a2(n) })
        eventually_predicate_nat(function(n: Nat) { a(n).abs <= c * b(n).abs })
        eventually_predicate_nat_has_witness(function(n: Nat) { a(n) = a2(n) })
        eventually_predicate_nat_has_witness(function(n: Nat) { a(n).abs <= c * b(n).abs })
        let n1: Nat satisfy {
            eventually_after_nat(function(n: Nat) { a(n) = a2(n) }, n1)
        }
        let n2: Nat satisfy {
            eventually_after_nat(function(n: Nat) { a(n).abs <= c * b(n).abs }, n2)
        }
        let n0 = n1.max(n2)
        max_imp_gte[Nat](n1, n2)
        n1.max(n2) >= n1 and n1.max(n2) >= n2
        n1 <= n0
        n2 <= n0
        forall(i: Nat) {
            if n0 <= i {
                lte_trans(n1, n0, i)
                lte_trans(n2, n0, i)
                eventually_after_nat_at(function(n: Nat) { a(n) = a2(n) }, n1, i)
                eventually_after_nat_at(function(n: Nat) { a(n).abs <= c * b(n).abs }, n2, i)
                a(i) = a2(i)
                a(i).abs <= c * b(i).abs
                a2(i).abs = a(i).abs
                a2(i).abs <= c * b(i).abs
                function(k: Nat) { a2(k).abs <= c * b(k).abs }(i)
            }
        }
        eventually_after_nat(function(k: Nat) { a2(k).abs <= c * b(k).abs }, n0)
        eventually_predicate_nat_intro(function(k: Nat) { a2(k).abs <= c * b(k).abs }, n0)
        eventually_predicate_nat(function(k: Nat) { a2(k).abs <= c * b(k).abs })
        c.is_positive
        big_o_seq_intro(a2, b, c)
        is_big_o_seq(a2, b)
    }
}

/// Big-O is preserved by eventual equality on the right sequence.
theorem big_o_seq_congr_right(a: Nat -> Real, b: Nat -> Real, b2: Nat -> Real) {
    eventually_seq_eq(b, b2) and is_big_o_seq(a, b) implies is_big_o_seq(a, b2)
} by {
    if eventually_seq_eq(b, b2) and is_big_o_seq(a, b) {
        let c: Real satisfy {
            c.is_positive and eventually_predicate_nat(function(n: Nat) { a(n).abs <= c * b(n).abs })
        }
        eventually_seq_eq(b, b2) = eventually_predicate_nat(function(n: Nat) { b(n) = b2(n) })
        eventually_predicate_nat(function(n: Nat) { b(n) = b2(n) })
        eventually_predicate_nat(function(n: Nat) { a(n).abs <= c * b(n).abs })
        eventually_predicate_nat_has_witness(function(n: Nat) { b(n) = b2(n) })
        eventually_predicate_nat_has_witness(function(n: Nat) { a(n).abs <= c * b(n).abs })
        let n1: Nat satisfy {
            eventually_after_nat(function(n: Nat) { b(n) = b2(n) }, n1)
        }
        let n2: Nat satisfy {
            eventually_after_nat(function(n: Nat) { a(n).abs <= c * b(n).abs }, n2)
        }
        let n0 = n1.max(n2)
        max_imp_gte[Nat](n1, n2)
        n1.max(n2) >= n1 and n1.max(n2) >= n2
        n1 <= n0
        n2 <= n0
        forall(i: Nat) {
            if n0 <= i {
                lte_trans(n1, n0, i)
                lte_trans(n2, n0, i)
                eventually_after_nat_at(function(n: Nat) { b(n) = b2(n) }, n1, i)
                eventually_after_nat_at(function(n: Nat) { a(n).abs <= c * b(n).abs }, n2, i)
                b(i) = b2(i)
                a(i).abs <= c * b(i).abs
                b(i).abs = b2(i).abs
                c * b(i).abs = c * b2(i).abs
                a(i).abs <= c * b2(i).abs
                function(k: Nat) { a(k).abs <= c * b2(k).abs }(i)
            }
        }
        eventually_after_nat(function(k: Nat) { a(k).abs <= c * b2(k).abs }, n0)
        eventually_predicate_nat_intro(function(k: Nat) { a(k).abs <= c * b2(k).abs }, n0)
        eventually_predicate_nat(function(k: Nat) { a(k).abs <= c * b2(k).abs })
        c.is_positive
        big_o_seq_intro(a, b2, c)
        is_big_o_seq(a, b2)
    }
}

/// Little-o is preserved by eventual equality on the left sequence.
theorem little_o_seq_congr_left(a: Nat -> Real, a2: Nat -> Real, b: Nat -> Real) {
    eventually_seq_eq(a, a2) and is_little_o_seq(a, b) implies is_little_o_seq(a2, b)
} by {
    if eventually_seq_eq(a, a2) and is_little_o_seq(a, b) {
        forall(eps: Real) {
            if eps.is_positive {
                is_little_o_seq(a, b) = forall(delta: Real) {
                    delta.is_positive implies eventually_predicate_nat(function(n: Nat) {
                        a(n).abs <= delta * b(n).abs
                    })
                }
                eventually_predicate_nat(function(n: Nat) { a(n).abs <= eps * b(n).abs })
                eventually_seq_eq(a, a2) = eventually_predicate_nat(function(n: Nat) { a(n) = a2(n) })
                eventually_predicate_nat(function(n: Nat) { a(n) = a2(n) })
                eventually_predicate_nat_has_witness(function(n: Nat) { a(n) = a2(n) })
                eventually_predicate_nat_has_witness(function(n: Nat) { a(n).abs <= eps * b(n).abs })
                let n1: Nat satisfy {
                    eventually_after_nat(function(n: Nat) { a(n) = a2(n) }, n1)
                }
                let n2: Nat satisfy {
                    eventually_after_nat(function(n: Nat) { a(n).abs <= eps * b(n).abs }, n2)
                }
                let n0 = n1.max(n2)
                max_imp_gte[Nat](n1, n2)
                n1.max(n2) >= n1 and n1.max(n2) >= n2
                n1 <= n0
                n2 <= n0
                forall(i: Nat) {
                    if n0 <= i {
                        lte_trans(n1, n0, i)
                        lte_trans(n2, n0, i)
                        eventually_after_nat_at(function(n: Nat) { a(n) = a2(n) }, n1, i)
                        eventually_after_nat_at(function(n: Nat) { a(n).abs <= eps * b(n).abs }, n2, i)
                        a(i) = a2(i)
                        a(i).abs <= eps * b(i).abs
                        a2(i).abs = a(i).abs
                        a2(i).abs <= eps * b(i).abs
                        function(k: Nat) { a2(k).abs <= eps * b(k).abs }(i)
                    }
                }
                eventually_after_nat(function(k: Nat) { a2(k).abs <= eps * b(k).abs }, n0)
                eventually_predicate_nat_intro(function(k: Nat) { a2(k).abs <= eps * b(k).abs }, n0)
                eventually_predicate_nat(function(k: Nat) { a2(k).abs <= eps * b(k).abs })
            }
        }
        is_little_o_seq(a2, b) = forall(eps: Real) {
            eps.is_positive implies eventually_predicate_nat(function(n: Nat) {
                a2(n).abs <= eps * b(n).abs
            })
        }
        is_little_o_seq(a2, b)
    }
}

/// Little-o is preserved by eventual equality on the right sequence.
theorem little_o_seq_congr_right(a: Nat -> Real, b: Nat -> Real, b2: Nat -> Real) {
    eventually_seq_eq(b, b2) and is_little_o_seq(a, b) implies is_little_o_seq(a, b2)
} by {
    if eventually_seq_eq(b, b2) and is_little_o_seq(a, b) {
        forall(eps: Real) {
            if eps.is_positive {
                is_little_o_seq(a, b) = forall(delta: Real) {
                    delta.is_positive implies eventually_predicate_nat(function(n: Nat) {
                        a(n).abs <= delta * b(n).abs
                    })
                }
                eventually_predicate_nat(function(n: Nat) { a(n).abs <= eps * b(n).abs })
                eventually_seq_eq(b, b2) = eventually_predicate_nat(function(n: Nat) { b(n) = b2(n) })
                eventually_predicate_nat(function(n: Nat) { b(n) = b2(n) })
                eventually_predicate_nat_has_witness(function(n: Nat) { b(n) = b2(n) })
                eventually_predicate_nat_has_witness(function(n: Nat) { a(n).abs <= eps * b(n).abs })
                let n1: Nat satisfy {
                    eventually_after_nat(function(n: Nat) { b(n) = b2(n) }, n1)
                }
                let n2: Nat satisfy {
                    eventually_after_nat(function(n: Nat) { a(n).abs <= eps * b(n).abs }, n2)
                }
                let n0 = n1.max(n2)
                max_imp_gte[Nat](n1, n2)
                n1.max(n2) >= n1 and n1.max(n2) >= n2
                n1 <= n0
                n2 <= n0
                forall(i: Nat) {
                    if n0 <= i {
                        lte_trans(n1, n0, i)
                        lte_trans(n2, n0, i)
                        eventually_after_nat_at(function(n: Nat) { b(n) = b2(n) }, n1, i)
                        eventually_after_nat_at(function(n: Nat) { a(n).abs <= eps * b(n).abs }, n2, i)
                        b(i) = b2(i)
                        a(i).abs <= eps * b(i).abs
                        b(i).abs = b2(i).abs
                        eps * b(i).abs = eps * b2(i).abs
                        a(i).abs <= eps * b2(i).abs
                        function(k: Nat) { a(k).abs <= eps * b2(k).abs }(i)
                    }
                }
                eventually_after_nat(function(k: Nat) { a(k).abs <= eps * b2(k).abs }, n0)
                eventually_predicate_nat_intro(function(k: Nat) { a(k).abs <= eps * b2(k).abs }, n0)
                eventually_predicate_nat(function(k: Nat) { a(k).abs <= eps * b2(k).abs })
            }
        }
        is_little_o_seq(a, b2) = forall(eps: Real) {
            eps.is_positive implies eventually_predicate_nat(function(n: Nat) {
                a(n).abs <= eps * b2(n).abs
            })
        }
        is_little_o_seq(a, b2)
    }
}
