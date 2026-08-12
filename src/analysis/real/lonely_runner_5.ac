/// The lonely runner conjecture, the five-runner case.
///
/// Runner 0 is the lonely runner, stationary at the origin. Five runners total
/// means four moving runners, at distinct positive integer speeds; the required
/// distance is `1/6` of the circumference. This file proves the concrete cases
/// with the smallest speed tuples (1,2,3,4), (1,2,3,5) and (1,2,3,6), and the
/// general case where all speeds lie within a factor of five (t = 1/(6a)
/// works).  It also develops the interval machinery of the classical proof —
/// the lonely intervals `[(6k+1)/(6v), (6k+5)/(6v)]` — and states the full
/// five-runner statement, whose remaining case (the largest speed more than
/// five times the smallest) is the classical case analysis, not yet proved.
from analysis.real.lonely_runner import frac, circular_distance, cd_bound, cd_bound_frac,
    from_nat_real_positive, same_denom_lte, from_nat_frac_lt_one,
    one_minus_frac, from_nat_real_lt, nat_lt_one_five, nat_lt_two_five, nat_lt_three_five,
    nat_zero_lt_five
from nat import Nat, from_nat, from_nat_add, from_nat_mul, from_nat_one, from_nat_zero,
    lt_add_suc, lt_imp_lte_suc, lte_and_lt, lt_and_lte, add_sub, add_comm, sub_pos, lte_ref,
    lt_mul_both, add_suc_right, add_one_right
from int import Int
from rat import Rat
from order import lt_imp_lte, lt_not_ref, lt_imp_ne, lte_min_of_bounds, lt_iff_lte_and_ne,
    lt_of_lt_of_lte, lte_trans
from real import Real, floor, lt_add_one, lt_lte_trans, lte_lt_trans, add_from_int,

    lt_add_left, lt_add_right, lte_add_right, lte_add_left,
    mul_one_left, mul_div_cancel, mul_one_over, div_cancel_common,
    sub_cancels, mul_sub_distrib_left, mul_inverse
from algebra.add_ordered_group import minus_flips_inequality, add_le_iff_le_sub_right
from ordered_field import mul_lt_mul_of_pos_right, inverse_of_positive_is_positive,
    inverse_on_positive_flips_inequality
from algebra.field.field import mul_not_zero
from analysis.real.real_base_digits import real_from_int_lte, floor_is_greatest, int_lt_imp_add_one_lte
from analysis.real.real_from_nat_order import from_nat_real_lte, div_le_div_pos, from_nat_real_nonnegative
from analysis.real.real_floor_div import floor_nat_div
from real import from_nat_is_from_rat
from data.basic.logic import exists_intro

numerals Real
numerals Nat

/// The embedding of the naturals into the integers, then into the reals, is the
/// natural embedding into the reals.
theorem real_from_int_from_nat(n: Nat) {
    Real.from_int(Int.from_nat(n)) = from_nat[Real](n)
} by {
    Real.from_int(Int.from_nat(n)) = Real.from_rat(Rat.from_int(Int.from_nat(n)))
    Real.from_rat(Rat.from_int(Int.from_nat(n))) = Real.from_rat(Rat.from_nat(n))
    Real.from_rat(Rat.from_nat(n)) = from_nat[Real](n)
    Real.from_int(Int.from_nat(n)) = from_nat[Real](n)
}

/// The floor of a number in `[k, k + 1)` is `k`.
///
/// The general form of `floor_zero_of_unit_interval`, with the interval shifted
/// by a natural number.
theorem floor_of_nat_interval(k: Nat, x: Real) {
    from_nat[Real](k) <= x and x < from_nat[Real](k) + Real.1
    implies floor(x) = Int.from_nat(k)
} by {
    if from_nat[Real](k) <= x and x < from_nat[Real](k) + Real.1 {
        real_from_int_from_nat(k)
        Real.from_int(Int.from_nat(k)) = from_nat[Real](k)
        (Real.from_int(Int.from_nat(k)) <= x)
        floor_is_greatest(x, Int.from_nat(k))
        Int.from_nat(k) <= floor(x)
        if Int.from_nat(k) + Int.1 <= floor(x) {
            real_from_int_lte(Int.from_nat(k) + Int.1, floor(x))
            (Real.from_int(Int.from_nat(k) + Int.1) <= Real.from_int(floor(x)))
            (Real.from_int(floor(x)) <= x)
            lte_trans(Real.from_int(Int.from_nat(k) + Int.1), Real.from_int(floor(x)), x)
            Real.from_int(Int.from_nat(k) + Int.1) <= x
            real_from_int_from_nat(k)
            Real.from_int(Int.from_nat(k)) = from_nat[Real](k)
            add_from_int(Int.from_nat(k), Int.1)
            (Real.from_int(Int.from_nat(k)) + Real.from_int(Int.1) = Real.from_int(Int.from_nat(k) + Int.1))
            (Real.from_int(Int.1) = Real.1)
            (Real.from_int(Int.from_nat(k)) + Real.1 = Real.from_int(Int.from_nat(k) + Int.1))
            (from_nat[Real](k) + Real.1 = Real.from_int(Int.from_nat(k) + Int.1))
            (Real.from_int(Int.from_nat(k) + Int.1) = from_nat[Real](k) + Real.1)
            (x < from_nat[Real](k) + Real.1)
            lt_lte_trans(x, from_nat[Real](k) + Real.1, Real.from_int(Int.from_nat(k) + Int.1))
            x < Real.from_int(Int.from_nat(k) + Int.1)
            lt_not_ref(Real.from_int(Int.from_nat(k) + Int.1))
            if x < Real.from_int(Int.from_nat(k) + Int.1) {
                lte_lt_trans(Real.from_int(Int.from_nat(k) + Int.1), x, Real.from_int(Int.from_nat(k) + Int.1))
                Real.from_int(Int.from_nat(k) + Int.1) < Real.from_int(Int.from_nat(k) + Int.1)
                false
            }
            false
        }
        not (Int.from_nat(k) + Int.1 <= floor(x))
        if floor(x) != Int.from_nat(k) {
            lt_iff_lte_and_ne(Int.from_nat(k), floor(x))
            (Int.from_nat(k) < floor(x)) = (Int.from_nat(k) <= floor(x) and Int.from_nat(k) != floor(x))
            Int.from_nat(k) < floor(x)
            int_lt_imp_add_one_lte(Int.from_nat(k), floor(x))
            Int.from_nat(k) + Int.1 <= floor(x)
            false
        }
        floor(x) = Int.from_nat(k)
    }
}

/// Six is positive.
theorem nat_zero_lt_six {
    Nat.0 < Nat.6
} by {
    lt_add_suc(Nat.0, Nat.5)
    Nat.0 < Nat.0 + Nat.5.suc
    (Nat.0 + Nat.5.suc = Nat.6)
    Nat.0 < Nat.6
}

/// One is below six.
theorem nat_lt_one_six {
    Nat.1 < Nat.6
} by {
    lt_add_suc(Nat.1, Nat.4)
    Nat.1 < Nat.1 + Nat.4.suc
    (Nat.1 + Nat.4.suc = Nat.6)
    Nat.1 < Nat.6
}

/// Two is below six.
theorem nat_lt_two_six {
    Nat.2 < Nat.6
} by {
    lt_add_suc(Nat.2, Nat.3)
    Nat.2 < Nat.2 + Nat.3.suc
    (Nat.2 + Nat.3.suc = Nat.6)
    Nat.2 < Nat.6
}

/// Three is below six.
theorem nat_lt_three_six {
    Nat.3 < Nat.6
} by {
    lt_add_suc(Nat.3, Nat.2)
    Nat.3 < Nat.3 + Nat.2.suc
    (Nat.3 + Nat.2.suc = Nat.6)
    Nat.3 < Nat.6
}

/// Four is below six.
theorem nat_lt_four_six {
    Nat.4 < Nat.6
} by {
    lt_add_suc(Nat.4, Nat.1)
    Nat.4 < Nat.4 + Nat.1.suc
    add_suc_right(Nat.4, Nat.1)
    Nat.4 + Nat.1.suc = (Nat.4 + Nat.1).suc
    add_one_right(Nat.4)
    Nat.4 + Nat.1 = Nat.4.suc
    (Nat.4 + Nat.1.suc = Nat.6)
    Nat.4 < Nat.6
}

/// Five is below six.
theorem nat_lt_five_six {
    Nat.5 < Nat.6
} by {
    lt_add_suc(Nat.5, Nat.0)
    Nat.5 < Nat.5 + Nat.0.suc
    (Nat.5 + Nat.0.suc = Nat.6)
    Nat.5 < Nat.6
}

/// Adding a nonnegative amount preserves the order.
theorem lte_add_pos(a: Real, b: Real) {
    Real.0 <= b implies a <= a + b
} by {
    if Real.0 <= b {
        lte_add_right(Real.0, b, a)
        Real.0 + a <= b + a
        a <= a + b
    }
}

/// Adding the same amount to both sides preserves the strict order.
theorem lt_add_pos(a: Real, b: Real) {
    Real.0 < b implies a < a + b
} by {
    if Real.0 < b {
        lt_add_right(Real.0, b, a)
        Real.0 + a < b + a
        a < a + b
    }
}

/// A smaller amount leaves a larger remainder.
theorem lte_sub_left(a: Real, b: Real, c: Real) {
    a <= b implies c - b <= c - a
} by {
    if a <= b {
        minus_flips_inequality(a, b)
        -b <= -a
        lte_add_right(-b, -a, c)
        c + -b <= c + -a
        c - b <= c - a
    }
}

/// A point in the middle of a unit interval is lonely.
///
/// The core of the interval analysis for the five-runner case: any point whose
/// fractional part lies between 1/6 and 5/6 is at circular distance at least
/// 1/6 from the origin.
theorem cd_of_interval_sixth(k: Nat, x: Real) {
    from_nat[Real](k) + Real.1 / from_nat[Real](Nat.6) <= x
        and x <= from_nat[Real](k) + from_nat[Real](Nat.5) / from_nat[Real](Nat.6)
    implies Real.1 / from_nat[Real](Nat.6) <= circular_distance(x)
} by {
    if from_nat[Real](k) + Real.1 / from_nat[Real](Nat.6) <= x
        and x <= from_nat[Real](k) + from_nat[Real](Nat.5) / from_nat[Real](Nat.6) {
        from_nat_real_positive(Nat.6)
        (Real.0 < from_nat[Real](Nat.6))
        from_nat_real_nonnegative(Nat.1)
        (Real.0 <= from_nat[Real](Nat.1))
        div_le_div_pos(Real.0, from_nat[Real](Nat.1), from_nat[Real](Nat.6))
        (Real.0 / from_nat[Real](Nat.6) <= from_nat[Real](Nat.1) / from_nat[Real](Nat.6))
        (Real.0 / from_nat[Real](Nat.6) = Real.0)
        (Real.0 <= from_nat[Real](Nat.1) / from_nat[Real](Nat.6))
        (from_nat[Real](Nat.1) = Real.1)
        Real.0 <= Real.1 / from_nat[Real](Nat.6)
        lte_add_pos(from_nat[Real](k), Real.1 / from_nat[Real](Nat.6))
        from_nat[Real](k) <= from_nat[Real](k) + Real.1 / from_nat[Real](Nat.6)
        lte_trans(from_nat[Real](k), from_nat[Real](k) + Real.1 / from_nat[Real](Nat.6), x)
        from_nat[Real](k) <= x
        from_nat_frac_lt_one(Nat.5, Nat.6)
        (from_nat[Real](Nat.5) / from_nat[Real](Nat.6) < Real.1)
        lt_add_left(from_nat[Real](Nat.5) / from_nat[Real](Nat.6), Real.1, from_nat[Real](k))
        (from_nat[Real](k) + from_nat[Real](Nat.5) / from_nat[Real](Nat.6)
            < from_nat[Real](k) + Real.1)
        lte_lt_trans(x, from_nat[Real](k) + from_nat[Real](Nat.5) / from_nat[Real](Nat.6),
            from_nat[Real](k) + Real.1)
        x < from_nat[Real](k) + Real.1
        floor_of_nat_interval(k, x)
        floor(x) = Int.from_nat(k)
        frac(x) = x - Real.from_int(floor(x))
        real_from_int_from_nat(k)
        (Real.from_int(Int.from_nat(k)) = from_nat[Real](k))
        frac(x) = x - from_nat[Real](k)
        (from_nat[Real](k) + Real.1 / from_nat[Real](Nat.6) <= x)
        add_le_iff_le_sub_right(Real.1 / from_nat[Real](Nat.6), from_nat[Real](k), x)
        (Real.1 / from_nat[Real](Nat.6) <= x - from_nat[Real](k))
        (Real.1 / from_nat[Real](Nat.6) <= frac(x))
        (x <= from_nat[Real](k) + from_nat[Real](Nat.5) / from_nat[Real](Nat.6))
        lte_add_right(x, from_nat[Real](k) + from_nat[Real](Nat.5) / from_nat[Real](Nat.6),
            -from_nat[Real](k))
        (x + -from_nat[Real](k)
            <= from_nat[Real](k) + from_nat[Real](Nat.5) / from_nat[Real](Nat.6) + -from_nat[Real](k))
        (x + -from_nat[Real](k) = x - from_nat[Real](k))
        sub_cancels(from_nat[Real](k), from_nat[Real](Nat.5) / from_nat[Real](Nat.6))
        (from_nat[Real](k) + from_nat[Real](Nat.5) / from_nat[Real](Nat.6) - from_nat[Real](k)
            = from_nat[Real](Nat.5) / from_nat[Real](Nat.6))
        (from_nat[Real](k) + from_nat[Real](Nat.5) / from_nat[Real](Nat.6) + -from_nat[Real](k)
            = from_nat[Real](Nat.5) / from_nat[Real](Nat.6))
        (x - from_nat[Real](k) <= from_nat[Real](Nat.5) / from_nat[Real](Nat.6))
        (frac(x) <= from_nat[Real](Nat.5) / from_nat[Real](Nat.6))
        lte_sub_left(frac(x), from_nat[Real](Nat.5) / from_nat[Real](Nat.6), Real.1)
        (Real.1 - from_nat[Real](Nat.5) / from_nat[Real](Nat.6) <= Real.1 - frac(x))
        one_minus_frac(Nat.5, Nat.6)
        (Real.1 - from_nat[Real](Nat.5) / from_nat[Real](Nat.6)
            = from_nat[Real](Nat.6 - Nat.5) / from_nat[Real](Nat.6))
        (Nat.6 - Nat.5 = Nat.1)
        (Real.1 - from_nat[Real](Nat.5) / from_nat[Real](Nat.6)
            = from_nat[Real](Nat.1) / from_nat[Real](Nat.6))
        (from_nat[Real](Nat.1) = Real.1)
        (Real.1 - from_nat[Real](Nat.5) / from_nat[Real](Nat.6)
            = Real.1 / from_nat[Real](Nat.6))
        (Real.1 / from_nat[Real](Nat.6) <= Real.1 - frac(x))
        lte_min_of_bounds(Real.1 / from_nat[Real](Nat.6), frac(x), Real.1 - frac(x))
        Real.1 / from_nat[Real](Nat.6) <= frac(x).min(Real.1 - frac(x))
        circular_distance(x) = frac(x).min(Real.1 - frac(x))
        Real.1 / from_nat[Real](Nat.6) <= circular_distance(x)
    }
}

/// Five runners (four moving, speeds 1, 2, 3 and 4): at t = 1/6 the runners are
/// at 1/6, 2/6, 3/6 and 4/6, all at distance at least 1/6 from the origin.
theorem lonely_runner_five_speeds_1234 {
    exists(t: Real) {
        Real.1 / from_nat[Real](Nat.6) <= circular_distance(Real.1 * t)
            and Real.1 / from_nat[Real](Nat.6) <= circular_distance(from_nat[Real](Nat.2) * t)
            and Real.1 / from_nat[Real](Nat.6) <= circular_distance(from_nat[Real](Nat.3) * t)
            and Real.1 / from_nat[Real](Nat.6) <= circular_distance(from_nat[Real](Nat.4) * t)
    }
} by {
    mul_one_left(Real.1 / from_nat[Real](Nat.6))
    (Real.1 * (Real.1 / from_nat[Real](Nat.6)) = Real.1 / from_nat[Real](Nat.6))
    cd_bound_frac(Nat.1, Nat.6)
    (Nat.1 <= Nat.1 and Nat.1 < Nat.6)
    (Real.1 / from_nat[Real](Nat.6) <= circular_distance(from_nat[Real](Nat.1) / from_nat[Real](Nat.6)))
    (from_nat[Real](Nat.1) = Real.1)
    (Real.1 / from_nat[Real](Nat.6) <= circular_distance(Real.1 / from_nat[Real](Nat.6)))
    (Real.1 / from_nat[Real](Nat.6) <= circular_distance(Real.1 * (Real.1 / from_nat[Real](Nat.6))))
    from_nat_real_positive(Nat.6)
    (Real.0 < from_nat[Real](Nat.6))
    (from_nat[Real](Nat.6) != Real.0)
    mul_one_over(from_nat[Real](Nat.2), from_nat[Real](Nat.6))
    (from_nat[Real](Nat.2) * (Real.1 / from_nat[Real](Nat.6)) = from_nat[Real](Nat.2) / from_nat[Real](Nat.6))
    cd_bound_frac(Nat.2, Nat.6)
    (Nat.1 <= Nat.2 and Nat.2 < Nat.6)
    (Real.1 / from_nat[Real](Nat.6) <= circular_distance(from_nat[Real](Nat.2) / from_nat[Real](Nat.6)))
    (Real.1 / from_nat[Real](Nat.6) <= circular_distance(from_nat[Real](Nat.2) * (Real.1 / from_nat[Real](Nat.6))))
    mul_one_over(from_nat[Real](Nat.3), from_nat[Real](Nat.6))
    (from_nat[Real](Nat.3) * (Real.1 / from_nat[Real](Nat.6)) = from_nat[Real](Nat.3) / from_nat[Real](Nat.6))
    cd_bound_frac(Nat.3, Nat.6)
    (Nat.1 <= Nat.3 and Nat.3 < Nat.6)
    (Real.1 / from_nat[Real](Nat.6) <= circular_distance(from_nat[Real](Nat.3) / from_nat[Real](Nat.6)))
    (Real.1 / from_nat[Real](Nat.6) <= circular_distance(from_nat[Real](Nat.3) * (Real.1 / from_nat[Real](Nat.6))))
    mul_one_over(from_nat[Real](Nat.4), from_nat[Real](Nat.6))
    (from_nat[Real](Nat.4) * (Real.1 / from_nat[Real](Nat.6)) = from_nat[Real](Nat.4) / from_nat[Real](Nat.6))
    cd_bound_frac(Nat.4, Nat.6)
    (Nat.1 <= Nat.4 and Nat.4 < Nat.6)
    (Real.1 / from_nat[Real](Nat.6) <= circular_distance(from_nat[Real](Nat.4) / from_nat[Real](Nat.6)))
    (Real.1 / from_nat[Real](Nat.6) <= circular_distance(from_nat[Real](Nat.4) * (Real.1 / from_nat[Real](Nat.6))))
    (Real.1 / from_nat[Real](Nat.6)
        <= circular_distance(Real.1 * (Real.1 / from_nat[Real](Nat.6)))
        and Real.1 / from_nat[Real](Nat.6)
            <= circular_distance(from_nat[Real](Nat.2) * (Real.1 / from_nat[Real](Nat.6)))
        and Real.1 / from_nat[Real](Nat.6)
            <= circular_distance(from_nat[Real](Nat.3) * (Real.1 / from_nat[Real](Nat.6)))
        and Real.1 / from_nat[Real](Nat.6)
            <= circular_distance(from_nat[Real](Nat.4) * (Real.1 / from_nat[Real](Nat.6))))
    exists(t: Real) {
        Real.1 / from_nat[Real](Nat.6) <= circular_distance(Real.1 * t)
            and Real.1 / from_nat[Real](Nat.6) <= circular_distance(from_nat[Real](Nat.2) * t)
            and Real.1 / from_nat[Real](Nat.6) <= circular_distance(from_nat[Real](Nat.3) * t)
            and Real.1 / from_nat[Real](Nat.6) <= circular_distance(from_nat[Real](Nat.4) * t)
    }
}

/// Five runners (four moving, speeds 1, 2, 3 and 5): at t = 1/6 the runners are
/// at 1/6, 2/6, 3/6 and 5/6, all at distance at least 1/6 from the origin.
theorem lonely_runner_five_speeds_1235 {
    exists(t: Real) {
        Real.1 / from_nat[Real](Nat.6) <= circular_distance(Real.1 * t)
            and Real.1 / from_nat[Real](Nat.6) <= circular_distance(from_nat[Real](Nat.2) * t)
            and Real.1 / from_nat[Real](Nat.6) <= circular_distance(from_nat[Real](Nat.3) * t)
            and Real.1 / from_nat[Real](Nat.6) <= circular_distance(from_nat[Real](Nat.5) * t)
    }
} by {
    mul_one_left(Real.1 / from_nat[Real](Nat.6))
    (Real.1 * (Real.1 / from_nat[Real](Nat.6)) = Real.1 / from_nat[Real](Nat.6))
    cd_bound_frac(Nat.1, Nat.6)
    (Nat.1 <= Nat.1 and Nat.1 < Nat.6)
    (Real.1 / from_nat[Real](Nat.6) <= circular_distance(from_nat[Real](Nat.1) / from_nat[Real](Nat.6)))
    (from_nat[Real](Nat.1) = Real.1)
    (Real.1 / from_nat[Real](Nat.6) <= circular_distance(Real.1 / from_nat[Real](Nat.6)))
    (Real.1 / from_nat[Real](Nat.6) <= circular_distance(Real.1 * (Real.1 / from_nat[Real](Nat.6))))
    from_nat_real_positive(Nat.6)
    (Real.0 < from_nat[Real](Nat.6))
    (from_nat[Real](Nat.6) != Real.0)
    mul_one_over(from_nat[Real](Nat.2), from_nat[Real](Nat.6))
    (from_nat[Real](Nat.2) * (Real.1 / from_nat[Real](Nat.6)) = from_nat[Real](Nat.2) / from_nat[Real](Nat.6))
    cd_bound_frac(Nat.2, Nat.6)
    (Nat.1 <= Nat.2 and Nat.2 < Nat.6)
    (Real.1 / from_nat[Real](Nat.6) <= circular_distance(from_nat[Real](Nat.2) / from_nat[Real](Nat.6)))
    (Real.1 / from_nat[Real](Nat.6) <= circular_distance(from_nat[Real](Nat.2) * (Real.1 / from_nat[Real](Nat.6))))
    mul_one_over(from_nat[Real](Nat.3), from_nat[Real](Nat.6))
    (from_nat[Real](Nat.3) * (Real.1 / from_nat[Real](Nat.6)) = from_nat[Real](Nat.3) / from_nat[Real](Nat.6))
    cd_bound_frac(Nat.3, Nat.6)
    (Nat.1 <= Nat.3 and Nat.3 < Nat.6)
    (Real.1 / from_nat[Real](Nat.6) <= circular_distance(from_nat[Real](Nat.3) / from_nat[Real](Nat.6)))
    (Real.1 / from_nat[Real](Nat.6) <= circular_distance(from_nat[Real](Nat.3) * (Real.1 / from_nat[Real](Nat.6))))
    mul_one_over(from_nat[Real](Nat.5), from_nat[Real](Nat.6))
    (from_nat[Real](Nat.5) * (Real.1 / from_nat[Real](Nat.6)) = from_nat[Real](Nat.5) / from_nat[Real](Nat.6))
    cd_bound_frac(Nat.5, Nat.6)
    nat_lt_one_five
    (Nat.1 < Nat.5)
    lt_imp_lte(Nat.1, Nat.5)
    Nat.1 <= Nat.5
    nat_lt_five_six
    (Nat.5 < Nat.6)
    (Nat.1 <= Nat.5 and Nat.5 < Nat.6)
    (Real.1 / from_nat[Real](Nat.6) <= circular_distance(from_nat[Real](Nat.5) / from_nat[Real](Nat.6)))
    (Real.1 / from_nat[Real](Nat.6) <= circular_distance(from_nat[Real](Nat.5) * (Real.1 / from_nat[Real](Nat.6))))
    (Real.1 / from_nat[Real](Nat.6)
        <= circular_distance(Real.1 * (Real.1 / from_nat[Real](Nat.6)))
        and Real.1 / from_nat[Real](Nat.6)
            <= circular_distance(from_nat[Real](Nat.2) * (Real.1 / from_nat[Real](Nat.6)))
        and Real.1 / from_nat[Real](Nat.6)
            <= circular_distance(from_nat[Real](Nat.3) * (Real.1 / from_nat[Real](Nat.6)))
        and Real.1 / from_nat[Real](Nat.6)
            <= circular_distance(from_nat[Real](Nat.5) * (Real.1 / from_nat[Real](Nat.6))))
    exists(t: Real) {
        Real.1 / from_nat[Real](Nat.6) <= circular_distance(Real.1 * t)
            and Real.1 / from_nat[Real](Nat.6) <= circular_distance(from_nat[Real](Nat.2) * t)
            and Real.1 / from_nat[Real](Nat.6) <= circular_distance(from_nat[Real](Nat.3) * t)
            and Real.1 / from_nat[Real](Nat.6) <= circular_distance(from_nat[Real](Nat.5) * t)
    }
}

/// The position of a runner with speed between `a` and `5a` at time `1/(6a)`
/// is at circular distance at least 1/6.
///
/// The general form of the small-range argument: when all speeds lie in
/// `[a, 5a]`, the single time `1/(6a)` makes every runner lonely.
theorem cd_bound_sixth_scale(a: Nat, v: Nat) {
    Nat.1 <= a and a <= v and v <= Nat.5 * a
    implies Real.1 / from_nat[Real](Nat.6) <= circular_distance(
        from_nat[Real](v) / (from_nat[Real](Nat.6) * from_nat[Real](a)))
} by {
    if Nat.1 <= a and a <= v and v <= Nat.5 * a {
        from_nat_real_positive(Nat.6)
        (Real.0 < from_nat[Real](Nat.6))
        (from_nat[Real](Nat.6) != Real.0)
        from_nat_real_positive(a)
        (Real.0 < from_nat[Real](a))
        (from_nat[Real](a) != Real.0)
        lt_add_suc(Nat.0, Nat.0)
        (Nat.0 < Nat.0 + Nat.0.suc)
        (Nat.0 + Nat.0.suc = Nat.1)
        Nat.0 < Nat.1
        lt_of_lt_of_lte(Nat.0, Nat.1, a)
        Nat.0 < a
        lt_mul_both(Nat.6, Nat.0, a)
        (Nat.6 != Nat.0 and Nat.0 < a)
        (Nat.6 * Nat.0 < Nat.6 * a)
        (Nat.6 * Nat.0 = Nat.0)
        Nat.0 < Nat.6 * a
        from_nat_real_positive(Nat.6 * a)
        (Real.0 < from_nat[Real](Nat.6 * a))
        from_nat_mul[Real](Nat.6, a)
        (from_nat[Real](Nat.6 * a) = from_nat[Real](Nat.6) * from_nat[Real](a))
        (Real.0 < from_nat[Real](Nat.6) * from_nat[Real](a))
        (from_nat[Real](Nat.6) * from_nat[Real](a) != Real.0)
        mul_one_over(from_nat[Real](v), from_nat[Real](Nat.6) * from_nat[Real](a))
        (from_nat[Real](v) * (Real.1 / (from_nat[Real](Nat.6) * from_nat[Real](a)))
            = from_nat[Real](v) / (from_nat[Real](Nat.6) * from_nat[Real](a)))
        div_cancel_common(Real.1, from_nat[Real](a), from_nat[Real](Nat.6))
        ((Real.1 * from_nat[Real](a)) / (from_nat[Real](Nat.6) * from_nat[Real](a))
            = Real.1 / from_nat[Real](Nat.6))
        (Real.1 * from_nat[Real](a) = from_nat[Real](a))
        (from_nat[Real](a) / (from_nat[Real](Nat.6) * from_nat[Real](a))
            = Real.1 / from_nat[Real](Nat.6))
        from_nat_real_lte(a, v)
        (from_nat[Real](a) <= from_nat[Real](v))
        div_le_div_pos(from_nat[Real](a), from_nat[Real](v),
            from_nat[Real](Nat.6) * from_nat[Real](a))
        (from_nat[Real](a) / (from_nat[Real](Nat.6) * from_nat[Real](a))
            <= from_nat[Real](v) / (from_nat[Real](Nat.6) * from_nat[Real](a)))
        (Real.1 / from_nat[Real](Nat.6)
            <= from_nat[Real](v) / (from_nat[Real](Nat.6) * from_nat[Real](a)))
        from_nat_mul[Real](Nat.5, a)
        (from_nat[Real](Nat.5 * a) = from_nat[Real](Nat.5) * from_nat[Real](a))
        from_nat_real_lte(v, Nat.5 * a)
        (from_nat[Real](v) <= from_nat[Real](Nat.5 * a))
        (from_nat[Real](v) <= from_nat[Real](Nat.5) * from_nat[Real](a))
        div_le_div_pos(from_nat[Real](v), from_nat[Real](Nat.5) * from_nat[Real](a),
            from_nat[Real](Nat.6) * from_nat[Real](a))
        (from_nat[Real](v) / (from_nat[Real](Nat.6) * from_nat[Real](a))
            <= (from_nat[Real](Nat.5) * from_nat[Real](a))
                / (from_nat[Real](Nat.6) * from_nat[Real](a)))
        div_cancel_common(from_nat[Real](Nat.5), from_nat[Real](a), from_nat[Real](Nat.6))
        ((from_nat[Real](Nat.5) * from_nat[Real](a))
            / (from_nat[Real](Nat.6) * from_nat[Real](a))
            = from_nat[Real](Nat.5) / from_nat[Real](Nat.6))
        (from_nat[Real](v) / (from_nat[Real](Nat.6) * from_nat[Real](a))
            <= from_nat[Real](Nat.5) / from_nat[Real](Nat.6))
        (from_nat[Real](Nat.0) = Real.0)
        (from_nat[Real](Nat.0) + Real.1 / from_nat[Real](Nat.6)
            <= from_nat[Real](v) / (from_nat[Real](Nat.6) * from_nat[Real](a)))
        (from_nat[Real](v) / (from_nat[Real](Nat.6) * from_nat[Real](a))
            <= from_nat[Real](Nat.0) + from_nat[Real](Nat.5) / from_nat[Real](Nat.6))
        cd_of_interval_sixth(Nat.0,
            from_nat[Real](v) / (from_nat[Real](Nat.6) * from_nat[Real](a)))
        (Real.1 / from_nat[Real](Nat.6) <= circular_distance(
            from_nat[Real](v) / (from_nat[Real](Nat.6) * from_nat[Real](a))))
    }
}

/// Five runners whose speeds all lie within a factor of five: at t = 1/(6a)
/// every moving runner is at circular distance at least 1/6 from the origin.
///
/// This is the general small-range case: when the largest speed is at most five
/// times the smallest, the single time `1/(6a)` works for all four runners.
theorem lonely_runner_five_small_range(a: Nat, b: Nat, c: Nat, d: Nat) {
    Nat.1 <= a and a < b and b < c and c < d and d <= Nat.5 * a
    implies exists(t: Real) {
        Real.1 / from_nat[Real](Nat.6) <= circular_distance(from_nat[Real](a) * t)
            and Real.1 / from_nat[Real](Nat.6) <= circular_distance(from_nat[Real](b) * t)
            and Real.1 / from_nat[Real](Nat.6) <= circular_distance(from_nat[Real](c) * t)
            and Real.1 / from_nat[Real](Nat.6) <= circular_distance(from_nat[Real](d) * t)
    }
} by {
    if Nat.1 <= a and a < b and b < c and c < d and d <= Nat.5 * a {
        (Nat.1 <= a)
        (a < b)
        lt_imp_lte(a, b)
        a <= b
        (b < c)
        lt_imp_lte(b, c)
        b <= c
        lte_trans(a, b, c)
        a <= c
        (c < d)
        lt_imp_lte(c, d)
        c <= d
        lte_trans(a, c, d)
        a <= d
        (d <= Nat.5 * a)
        lte_trans(a, d, Nat.5 * a)
        a <= Nat.5 * a
        cd_bound_sixth_scale(a, a)
        (Nat.1 <= a and a <= a and a <= Nat.5 * a)
        (Real.1 / from_nat[Real](Nat.6) <= circular_distance(
            from_nat[Real](a) / (from_nat[Real](Nat.6) * from_nat[Real](a))))
        from_nat_real_positive(Nat.6)
        (Real.0 < from_nat[Real](Nat.6))
        (from_nat[Real](Nat.6) != Real.0)
        from_nat_real_positive(a)
        (Real.0 < from_nat[Real](a))
        (from_nat[Real](a) != Real.0)
        mul_one_over(from_nat[Real](a), from_nat[Real](Nat.6) * from_nat[Real](a))
        (from_nat[Real](a) * (Real.1 / (from_nat[Real](Nat.6) * from_nat[Real](a)))
            = from_nat[Real](a) / (from_nat[Real](Nat.6) * from_nat[Real](a)))
        (Real.1 / from_nat[Real](Nat.6) <= circular_distance(
            from_nat[Real](a) * (Real.1 / (from_nat[Real](Nat.6) * from_nat[Real](a)))))
        (Nat.1 <= a)
        (a <= b)
        (b <= c)
        (c <= d)
        lte_trans(c, d, Nat.5 * a)
        c <= Nat.5 * a
        lte_trans(b, c, Nat.5 * a)
        b <= Nat.5 * a
        cd_bound_sixth_scale(a, b)
        (Nat.1 <= a and a <= b and b <= Nat.5 * a)
        (Real.1 / from_nat[Real](Nat.6) <= circular_distance(
            from_nat[Real](b) / (from_nat[Real](Nat.6) * from_nat[Real](a))))
        mul_one_over(from_nat[Real](b), from_nat[Real](Nat.6) * from_nat[Real](a))
        (from_nat[Real](b) * (Real.1 / (from_nat[Real](Nat.6) * from_nat[Real](a)))
            = from_nat[Real](b) / (from_nat[Real](Nat.6) * from_nat[Real](a)))
        (Real.1 / from_nat[Real](Nat.6) <= circular_distance(
            from_nat[Real](b) * (Real.1 / (from_nat[Real](Nat.6) * from_nat[Real](a)))))
        (Nat.1 <= a)
        (a <= c)
        (c <= d)
        lte_trans(c, d, Nat.5 * a)
        c <= Nat.5 * a
        cd_bound_sixth_scale(a, c)
        (Nat.1 <= a and a <= c and c <= Nat.5 * a)
        (Real.1 / from_nat[Real](Nat.6) <= circular_distance(
            from_nat[Real](c) / (from_nat[Real](Nat.6) * from_nat[Real](a))))
        mul_one_over(from_nat[Real](c), from_nat[Real](Nat.6) * from_nat[Real](a))
        (from_nat[Real](c) * (Real.1 / (from_nat[Real](Nat.6) * from_nat[Real](a)))
            = from_nat[Real](c) / (from_nat[Real](Nat.6) * from_nat[Real](a)))
        (Real.1 / from_nat[Real](Nat.6) <= circular_distance(
            from_nat[Real](c) * (Real.1 / (from_nat[Real](Nat.6) * from_nat[Real](a)))))
        (Nat.1 <= a)
        (a <= d)
        (d <= Nat.5 * a)
        cd_bound_sixth_scale(a, d)
        (Nat.1 <= a and a <= d and d <= Nat.5 * a)
        (Real.1 / from_nat[Real](Nat.6) <= circular_distance(
            from_nat[Real](d) / (from_nat[Real](Nat.6) * from_nat[Real](a))))
        mul_one_over(from_nat[Real](d), from_nat[Real](Nat.6) * from_nat[Real](a))
        (from_nat[Real](d) * (Real.1 / (from_nat[Real](Nat.6) * from_nat[Real](a)))
            = from_nat[Real](d) / (from_nat[Real](Nat.6) * from_nat[Real](a)))
        (Real.1 / from_nat[Real](Nat.6) <= circular_distance(
            from_nat[Real](d) * (Real.1 / (from_nat[Real](Nat.6) * from_nat[Real](a)))))
        (Real.1 / from_nat[Real](Nat.6)
            <= circular_distance(from_nat[Real](a) * (Real.1 / (from_nat[Real](Nat.6) * from_nat[Real](a))))
            and Real.1 / from_nat[Real](Nat.6)
                <= circular_distance(from_nat[Real](b) * (Real.1 / (from_nat[Real](Nat.6) * from_nat[Real](a))))
            and Real.1 / from_nat[Real](Nat.6)
                <= circular_distance(from_nat[Real](c) * (Real.1 / (from_nat[Real](Nat.6) * from_nat[Real](a))))
            and Real.1 / from_nat[Real](Nat.6)
                <= circular_distance(from_nat[Real](d) * (Real.1 / (from_nat[Real](Nat.6) * from_nat[Real](a)))))
        exists(t: Real) {
            Real.1 / from_nat[Real](Nat.6) <= circular_distance(from_nat[Real](a) * t)
                and Real.1 / from_nat[Real](Nat.6) <= circular_distance(from_nat[Real](b) * t)
                and Real.1 / from_nat[Real](Nat.6) <= circular_distance(from_nat[Real](c) * t)
                and Real.1 / from_nat[Real](Nat.6) <= circular_distance(from_nat[Real](d) * t)
        }
    }
}

/// Division distributes over addition when the denominator is nonzero.
theorem div_add_distrib_local(a: Real, b: Real, c: Real) {
    c != Real.0 implies (a + b) / c = a / c + b / c
} by {
    if c != Real.0 {
        (a + b) * c.inverse = a * c.inverse + b * c.inverse
        ((a + b) / c = (a + b) * c.inverse)
        (a / c = a * c.inverse)
        (b / c = b * c.inverse)
        (a + b) / c = a / c + b / c
    }
}

/// One fifth is at most five sixths.
theorem fifth_le_five_sixths {
    Real.1 / from_nat[Real](Nat.5) <= from_nat[Real](Nat.5) / from_nat[Real](Nat.6)
} by {
    nat_lt_two_five
    (Nat.2 < Nat.5)
    from_nat_real_lt(Nat.2, Nat.5)
    (from_nat[Real](Nat.2) < from_nat[Real](Nat.5))
    from_nat_real_positive(Nat.2)
    (Real.0 < from_nat[Real](Nat.2))
    from_nat_real_positive(Nat.5)
    (Real.0 < from_nat[Real](Nat.5))
    inverse_on_positive_flips_inequality(from_nat[Real](Nat.2), from_nat[Real](Nat.5))
    (from_nat[Real](Nat.5).inverse < from_nat[Real](Nat.2).inverse)
    (from_nat[Real](Nat.5).inverse = Real.1 / from_nat[Real](Nat.5))
    (from_nat[Real](Nat.2).inverse = Real.1 / from_nat[Real](Nat.2))
    (Real.1 / from_nat[Real](Nat.5) < Real.1 / from_nat[Real](Nat.2))
    lt_imp_lte(Real.1 / from_nat[Real](Nat.5), Real.1 / from_nat[Real](Nat.2))
    Real.1 / from_nat[Real](Nat.5) <= Real.1 / from_nat[Real](Nat.2)
    nat_lt_three_five
    (Nat.3 < Nat.5)
    from_nat_real_lte(Nat.3, Nat.5)
    (from_nat[Real](Nat.3) <= from_nat[Real](Nat.5))
    from_nat_real_positive(Nat.6)
    (Real.0 < from_nat[Real](Nat.6))
    div_le_div_pos(from_nat[Real](Nat.3), from_nat[Real](Nat.5), from_nat[Real](Nat.6))
    (from_nat[Real](Nat.3) / from_nat[Real](Nat.6) <= from_nat[Real](Nat.5) / from_nat[Real](Nat.6))
    from_nat_mul[Real](Nat.2, Nat.3)
    (from_nat[Real](Nat.2 * Nat.3) = from_nat[Real](Nat.2) * from_nat[Real](Nat.3))
    (Nat.2 * Nat.3 = Nat.6)
    (from_nat[Real](Nat.6) = from_nat[Real](Nat.2) * from_nat[Real](Nat.3))
    (from_nat[Real](Nat.6) != Real.0)
    from_nat_real_positive(Nat.2)
    (Real.0 < from_nat[Real](Nat.2))
    (from_nat[Real](Nat.2) != Real.0)
    (from_nat[Real](Nat.3) != Real.0)
    div_cancel_common(Real.1, from_nat[Real](Nat.3), from_nat[Real](Nat.2))
    ((Real.1 * from_nat[Real](Nat.3)) / (from_nat[Real](Nat.2) * from_nat[Real](Nat.3))
        = Real.1 / from_nat[Real](Nat.2))
    (Real.1 * from_nat[Real](Nat.3) = from_nat[Real](Nat.3))
    (from_nat[Real](Nat.3) / (from_nat[Real](Nat.2) * from_nat[Real](Nat.3))
        = Real.1 / from_nat[Real](Nat.2))
    (from_nat[Real](Nat.3) / from_nat[Real](Nat.6) = Real.1 / from_nat[Real](Nat.2))
    (Real.1 / from_nat[Real](Nat.2) <= from_nat[Real](Nat.5) / from_nat[Real](Nat.6))
    lte_trans(Real.1 / from_nat[Real](Nat.5), Real.1 / from_nat[Real](Nat.2),
        from_nat[Real](Nat.5) / from_nat[Real](Nat.6))
    Real.1 / from_nat[Real](Nat.5) <= from_nat[Real](Nat.5) / from_nat[Real](Nat.6)
}

/// Five runners (four moving, speeds 1, 2, 3 and 6): at t = 1/5 the runners are
/// at 1/5, 2/5, 3/5 and 6/5, and the last wraps to 1/5, so all are at distance
/// at least 1/6 from the origin.  The time 1/6 fails here (runner 6 returns to
/// the origin), so this case exercises the interval machinery.
theorem lonely_runner_five_speeds_1236 {
    exists(t: Real) {
        Real.1 / from_nat[Real](Nat.6) <= circular_distance(Real.1 * t)
            and Real.1 / from_nat[Real](Nat.6) <= circular_distance(from_nat[Real](Nat.2) * t)
            and Real.1 / from_nat[Real](Nat.6) <= circular_distance(from_nat[Real](Nat.3) * t)
            and Real.1 / from_nat[Real](Nat.6) <= circular_distance(from_nat[Real](Nat.6) * t)
    }
} by {
    mul_one_left(Real.1 / from_nat[Real](Nat.5))
    (Real.1 * (Real.1 / from_nat[Real](Nat.5)) = Real.1 / from_nat[Real](Nat.5))
    cd_bound_frac(Nat.1, Nat.5)
    (Nat.1 <= Nat.1 and Nat.1 < Nat.5)
    (Real.1 / from_nat[Real](Nat.5) <= circular_distance(from_nat[Real](Nat.1) / from_nat[Real](Nat.5)))
    (from_nat[Real](Nat.1) = Real.1)
    (Real.1 / from_nat[Real](Nat.5) <= circular_distance(Real.1 / from_nat[Real](Nat.5)))
    (Real.1 / from_nat[Real](Nat.5) <= circular_distance(Real.1 * (Real.1 / from_nat[Real](Nat.5))))
    cd_bound_frac(Nat.2, Nat.5)
    (Nat.1 <= Nat.2 and Nat.2 < Nat.5)
    (Real.1 / from_nat[Real](Nat.5) <= circular_distance(from_nat[Real](Nat.2) / from_nat[Real](Nat.5)))
    from_nat_real_positive(Nat.5)
    (Real.0 < from_nat[Real](Nat.5))
    (from_nat[Real](Nat.5) != Real.0)
    mul_one_over(from_nat[Real](Nat.2), from_nat[Real](Nat.5))
    (from_nat[Real](Nat.2) * (Real.1 / from_nat[Real](Nat.5)) = from_nat[Real](Nat.2) / from_nat[Real](Nat.5))
    (Real.1 / from_nat[Real](Nat.5) <= circular_distance(from_nat[Real](Nat.2) * (Real.1 / from_nat[Real](Nat.5))))
    cd_bound_frac(Nat.3, Nat.5)
    (Nat.1 <= Nat.3 and Nat.3 < Nat.5)
    (Real.1 / from_nat[Real](Nat.5) <= circular_distance(from_nat[Real](Nat.3) / from_nat[Real](Nat.5)))
    mul_one_over(from_nat[Real](Nat.3), from_nat[Real](Nat.5))
    (from_nat[Real](Nat.3) * (Real.1 / from_nat[Real](Nat.5)) = from_nat[Real](Nat.3) / from_nat[Real](Nat.5))
    (Real.1 / from_nat[Real](Nat.5) <= circular_distance(from_nat[Real](Nat.3) * (Real.1 / from_nat[Real](Nat.5))))
    // Runner 6 at t = 1/5 sits at 6/5, whose fractional part is 1/5.
    from_nat_real_positive(Nat.6)
    (Real.0 < from_nat[Real](Nat.6))
    (from_nat[Real](Nat.6) != Real.0)
    from_nat_real_positive(Nat.5)
    (Real.0 < from_nat[Real](Nat.5))
    (from_nat[Real](Nat.5) != Real.0)
    mul_one_over(from_nat[Real](Nat.6), from_nat[Real](Nat.5))
    (from_nat[Real](Nat.6) * (Real.1 / from_nat[Real](Nat.5)) = from_nat[Real](Nat.6) / from_nat[Real](Nat.5))
    (Nat.6 = Nat.5 + Nat.1)
    from_nat_add[Real](Nat.5, Nat.1)
    (from_nat[Real](Nat.5 + Nat.1) = from_nat[Real](Nat.5) + from_nat[Real](Nat.1))
    (from_nat[Real](Nat.6) = from_nat[Real](Nat.5) + from_nat[Real](Nat.1))
    div_add_distrib_local(from_nat[Real](Nat.5), from_nat[Real](Nat.1), from_nat[Real](Nat.5))
    ((from_nat[Real](Nat.5) + from_nat[Real](Nat.1)) / from_nat[Real](Nat.5)
        = from_nat[Real](Nat.5) / from_nat[Real](Nat.5) + from_nat[Real](Nat.1) / from_nat[Real](Nat.5))
    (from_nat[Real](Nat.6) / from_nat[Real](Nat.5)
        = from_nat[Real](Nat.5) / from_nat[Real](Nat.5) + from_nat[Real](Nat.1) / from_nat[Real](Nat.5))
    (from_nat[Real](Nat.5) != Real.0)
    mul_div_cancel(Real.1, from_nat[Real](Nat.5))
    (from_nat[Real](Nat.5) * (Real.1 / from_nat[Real](Nat.5)) = Real.1)
    mul_one_over(from_nat[Real](Nat.5), from_nat[Real](Nat.5))
    (from_nat[Real](Nat.5) * (Real.1 / from_nat[Real](Nat.5)) = from_nat[Real](Nat.5) / from_nat[Real](Nat.5))
    (from_nat[Real](Nat.5) / from_nat[Real](Nat.5) = Real.1)
    (from_nat[Real](Nat.6) / from_nat[Real](Nat.5)
        = Real.1 + from_nat[Real](Nat.1) / from_nat[Real](Nat.5))
    (from_nat[Real](Nat.1) = Real.1)
    (from_nat[Real](Nat.6) / from_nat[Real](Nat.5) = Real.1 + Real.1 / from_nat[Real](Nat.5))
    (from_nat[Real](Nat.6) / from_nat[Real](Nat.5) = from_nat[Real](Nat.1) + Real.1 / from_nat[Real](Nat.5))
    // The fractional part 6/5 - 1 = 1/5 lies in [1/6, 5/6].
    from_nat_real_lt(Nat.5, Nat.6)
    (from_nat[Real](Nat.5) < from_nat[Real](Nat.6))
    from_nat_real_positive(Nat.5)
    (Real.0 < from_nat[Real](Nat.5))
    from_nat_real_positive(Nat.6)
    (Real.0 < from_nat[Real](Nat.6))
    inverse_on_positive_flips_inequality(from_nat[Real](Nat.5), from_nat[Real](Nat.6))
    (from_nat[Real](Nat.6).inverse < from_nat[Real](Nat.5).inverse)
    (from_nat[Real](Nat.6).inverse = Real.1 / from_nat[Real](Nat.6))
    (from_nat[Real](Nat.5).inverse = Real.1 / from_nat[Real](Nat.5))
    (Real.1 / from_nat[Real](Nat.6) < Real.1 / from_nat[Real](Nat.5))
    lt_imp_lte(Real.1 / from_nat[Real](Nat.6), Real.1 / from_nat[Real](Nat.5))
    Real.1 / from_nat[Real](Nat.6) <= Real.1 / from_nat[Real](Nat.5)
    from_nat_real_lte(Nat.1, Nat.5)
    (from_nat[Real](Nat.1) <= from_nat[Real](Nat.5))
    (from_nat[Real](Nat.1) = Real.1)
    div_le_div_pos(Real.1, from_nat[Real](Nat.5), from_nat[Real](Nat.6))
    (Real.1 / from_nat[Real](Nat.6) <= from_nat[Real](Nat.5) / from_nat[Real](Nat.6))
    (from_nat[Real](Nat.5) / from_nat[Real](Nat.6) = from_nat[Real](Nat.5) / from_nat[Real](Nat.6))
    (Real.1 / from_nat[Real](Nat.6) <= from_nat[Real](Nat.5) / from_nat[Real](Nat.6))
    (from_nat[Real](Nat.6) / from_nat[Real](Nat.5) = from_nat[Real](Nat.1) + Real.1 / from_nat[Real](Nat.5))
    (from_nat[Real](Nat.1) + Real.1 / from_nat[Real](Nat.6)
        <= from_nat[Real](Nat.1) + Real.1 / from_nat[Real](Nat.5))
    lte_add_right(Real.1 / from_nat[Real](Nat.6), Real.1 / from_nat[Real](Nat.5),
        from_nat[Real](Nat.1))
    (from_nat[Real](Nat.1) + Real.1 / from_nat[Real](Nat.6)
        <= from_nat[Real](Nat.1) + Real.1 / from_nat[Real](Nat.5))
    (from_nat[Real](Nat.1) + Real.1 / from_nat[Real](Nat.6)
        <= from_nat[Real](Nat.6) / from_nat[Real](Nat.5))
    (from_nat[Real](Nat.5) / from_nat[Real](Nat.6) <= from_nat[Real](Nat.5) / from_nat[Real](Nat.6))
    (Real.1 / from_nat[Real](Nat.6) <= from_nat[Real](Nat.5) / from_nat[Real](Nat.6))
    lte_add_right(Real.1 / from_nat[Real](Nat.5), from_nat[Real](Nat.5) / from_nat[Real](Nat.6),
        from_nat[Real](Nat.1))
    (from_nat[Real](Nat.1) + Real.1 / from_nat[Real](Nat.5)
        <= from_nat[Real](Nat.1) + from_nat[Real](Nat.5) / from_nat[Real](Nat.6))
    (from_nat[Real](Nat.6) / from_nat[Real](Nat.5)
        <= from_nat[Real](Nat.1) + from_nat[Real](Nat.5) / from_nat[Real](Nat.6))
    cd_of_interval_sixth(Nat.1, from_nat[Real](Nat.6) / from_nat[Real](Nat.5))
    (Real.1 / from_nat[Real](Nat.6) <= circular_distance(from_nat[Real](Nat.6) / from_nat[Real](Nat.5)))
    (Real.1 / from_nat[Real](Nat.6) <= circular_distance(from_nat[Real](Nat.6) * (Real.1 / from_nat[Real](Nat.5))))
    // Chain all four together: 1/6 <= 1/5 <= each circular distance.
    from_nat_real_lt(Nat.5, Nat.6)
    (from_nat[Real](Nat.5) < from_nat[Real](Nat.6))
    from_nat_real_positive(Nat.5)
    (Real.0 < from_nat[Real](Nat.5))
    from_nat_real_positive(Nat.6)
    (Real.0 < from_nat[Real](Nat.6))
    inverse_on_positive_flips_inequality(from_nat[Real](Nat.5), from_nat[Real](Nat.6))
    (from_nat[Real](Nat.6).inverse < from_nat[Real](Nat.5).inverse)
    (from_nat[Real](Nat.6).inverse = Real.1 / from_nat[Real](Nat.6))
    (from_nat[Real](Nat.5).inverse = Real.1 / from_nat[Real](Nat.5))
    (Real.1 / from_nat[Real](Nat.6) < Real.1 / from_nat[Real](Nat.5))
    lt_imp_lte(Real.1 / from_nat[Real](Nat.6), Real.1 / from_nat[Real](Nat.5))
    (Real.1 / from_nat[Real](Nat.6) <= Real.1 / from_nat[Real](Nat.5))
    lte_trans(Real.1 / from_nat[Real](Nat.6), Real.1 / from_nat[Real](Nat.5),
        circular_distance(Real.1 * (Real.1 / from_nat[Real](Nat.5))))
    Real.1 / from_nat[Real](Nat.6) <= circular_distance(Real.1 * (Real.1 / from_nat[Real](Nat.5)))
    lte_trans(Real.1 / from_nat[Real](Nat.6), Real.1 / from_nat[Real](Nat.5),
        circular_distance(from_nat[Real](Nat.2) * (Real.1 / from_nat[Real](Nat.5))))
    Real.1 / from_nat[Real](Nat.6) <= circular_distance(from_nat[Real](Nat.2) * (Real.1 / from_nat[Real](Nat.5)))
    lte_trans(Real.1 / from_nat[Real](Nat.6), Real.1 / from_nat[Real](Nat.5),
        circular_distance(from_nat[Real](Nat.3) * (Real.1 / from_nat[Real](Nat.5))))
    Real.1 / from_nat[Real](Nat.6) <= circular_distance(from_nat[Real](Nat.3) * (Real.1 / from_nat[Real](Nat.5)))
    (Real.1 / from_nat[Real](Nat.6)
        <= circular_distance(Real.1 * (Real.1 / from_nat[Real](Nat.5)))
        and Real.1 / from_nat[Real](Nat.6)
            <= circular_distance(from_nat[Real](Nat.2) * (Real.1 / from_nat[Real](Nat.5)))
        and Real.1 / from_nat[Real](Nat.6)
            <= circular_distance(from_nat[Real](Nat.3) * (Real.1 / from_nat[Real](Nat.5)))
        and Real.1 / from_nat[Real](Nat.6)
            <= circular_distance(from_nat[Real](Nat.6) * (Real.1 / from_nat[Real](Nat.5))))
    exists(t: Real) {
        Real.1 / from_nat[Real](Nat.6) <= circular_distance(Real.1 * t)
            and Real.1 / from_nat[Real](Nat.6) <= circular_distance(from_nat[Real](Nat.2) * t)
            and Real.1 / from_nat[Real](Nat.6) <= circular_distance(from_nat[Real](Nat.3) * t)
            and Real.1 / from_nat[Real](Nat.6) <= circular_distance(from_nat[Real](Nat.6) * t)
    }
}

// ---- the general five-runner statement ----
//
// The general statement for five runners (four moving): for any four distinct
// positive integer speeds, there is a time at which all four moving runners are
// at circular distance at least 1/6 from the origin.
//
// The interval analysis behind the classical proof: for a runner with speed v,
// the lonely set L_v = {t : 1/6 <= cd(v t)} is a union of the intervals
// [(6k+1)/(6v), (6k+5)/(6v)] for integer k.  The theorem holds exactly when
// the intersection of the four lonely sets is nonempty.  `cd_of_interval_sixth`
// is the membership lemma for these intervals: a point whose fractional part
// lies between 1/6 and 5/6 is lonely.
//
// The small-range theorem `lonely_runner_five_small_range` above proves the
// case where the largest speed is at most five times the smallest (t = 1/(6a)
// works).  The remaining case, where the largest speed is more than five times
// the smallest, needs a case analysis on the relative sizes of the speeds (the
// classical n = 5 case analysis); it is stated here but not yet proved.
//
// theorem lonely_runner_five_general(a: Nat, b: Nat, c: Nat, d: Nat) {
//     Nat.1 <= a and a < b and b < c and c < d
//     implies exists(t: Real) {
//         Real.1 / from_nat[Real](Nat.6) <= circular_distance(from_nat[Real](a) * t)
//             and Real.1 / from_nat[Real](Nat.6) <= circular_distance(from_nat[Real](b) * t)
//             and Real.1 / from_nat[Real](Nat.6) <= circular_distance(from_nat[Real](c) * t)
//             and Real.1 / from_nat[Real](Nat.6) <= circular_distance(from_nat[Real](d) * t)
//     }
// } by {
//     // Split on whether the speeds are within a factor of five.
//     // If d <= 5a: t = 1/(6a) works (lonely_runner_five_small_range).
//     // If d > 5a: interval analysis on the lonely sets; open case in this file.
// }
