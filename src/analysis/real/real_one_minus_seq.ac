from nat import Nat
from real import Real, lte_antisymm, lte_trans, is_upper_bound, is_lower_bound,
    add_seq, converges, converges_to, limit, lte_add_right,
    converges_to_imp_converges, converges_to_unique, convergent_converges_to_limit,
    const_converges, const_limit, limit_add_seq
from data.basic.functions import function_extensionality
from algebra.add_comm_monoid_rearrange import add_swap_inner
from analysis.real.real_tail_supremum import tail_sup, tail_sup_bounds_terms, tail_sup_is_least,
    is_bounded_above, is_bounded_above_intro, is_bounded_above_witness
from analysis.real.real_limsup import limsup, neg_reverses_lte, is_bounded_below, is_bounded_below_intro,
    is_bounded_below_witness, neg_tail_sup, neg_tail_sup_converges
from analysis.real.real_liminf import liminf, tail_inf, tail_inf_bounds_terms, tail_inf_is_greatest,
    neg_seq, neg_seq_bounded_above, neg_seq_bounded_below, liminf_eq_limit_tail_inf,
    neg_tail_sup_neg_seq_eq_tail_inf

/// Reflection in one exchanges the two sides of an inequality.
theorem one_minus_flip(x: Real, y: Real) {
    Real.1 - x <= y implies Real.1 - y <= x
} by {
    if Real.1 - x <= y {
        lte_add_right(Real.1 - x, y, x)
        (Real.1 - x) + x <= y + x
        (Real.1 - x = Real.1 + -x)
        ((Real.1 + -x) + x = Real.1 + (-x + x))
        (-x + x = Real.0)
        (Real.1 + Real.0 = Real.1)
        ((Real.1 - x) + x = Real.1)
        Real.1 <= y + x
        lte_add_right(Real.1, y + x, -y)
        Real.1 + -y <= (y + x) + -y
        add_swap_inner[Real](y, x, -y, Real.0)
        ((y + x) + (-y + Real.0) = (y + -y) + (x + Real.0))
        (-y + Real.0 = -y)
        (x + Real.0 = x)
        (y + -y = Real.0)
        (Real.0 + x = x)
        ((y + x) + -y = x)
        (Real.1 + -y = Real.1 - y)
        Real.1 - y <= x
    }
}

/// The reflection of a sequence in one.
///
/// The shape a complement takes: a fraction and the fraction of the complement add to one.
define one_minus_seq(a: Nat -> Real) -> (Nat -> Real) {
    function(n: Nat) {
        Real.1 - a(n)
    }
}

/// Reflecting a bounded sequence leaves it bounded above.
theorem one_minus_bounded_above(a: Nat -> Real) {
    is_bounded_below(a) implies is_bounded_above(one_minus_seq(a))
} by {
    if is_bounded_below(a) {
        is_bounded_below_witness(a)
        exists(b: Real) { is_lower_bound(a, b) }
        let (b: Real) satisfy {
            is_lower_bound(a, b)
        }
        (is_lower_bound(a, b) = forall(k: Nat) { b <= a(k) })
        forall(n: Nat) {
            b <= a(n)
            neg_reverses_lte(b, a(n))
            -a(n) <= -b
            (Real.1 + -a(n) <= Real.1 + -b)
            (Real.1 - a(n) = Real.1 + -a(n))
            (Real.1 - b = Real.1 + -b)
            (one_minus_seq(a)(n) = Real.1 - a(n))
            one_minus_seq(a)(n) <= Real.1 - b
        }
        (is_upper_bound(one_minus_seq(a), Real.1 - b) = forall(n: Nat) {
            one_minus_seq(a)(n) <= Real.1 - b
        })
        is_upper_bound(one_minus_seq(a), Real.1 - b)
        is_bounded_above_intro(one_minus_seq(a), Real.1 - b)
        is_bounded_above(one_minus_seq(a))
    }
}

/// Reflecting a bounded sequence leaves it bounded below.
theorem one_minus_bounded_below(a: Nat -> Real) {
    is_bounded_above(a) implies is_bounded_below(one_minus_seq(a))
} by {
    if is_bounded_above(a) {
        is_bounded_above_witness(a)
        exists(b: Real) { is_upper_bound(a, b) }
        let (b: Real) satisfy {
            is_upper_bound(a, b)
        }
        (is_upper_bound(a, b) = forall(k: Nat) { a(k) <= b })
        forall(n: Nat) {
            a(n) <= b
            neg_reverses_lte(a(n), b)
            -b <= -a(n)
            (Real.1 + -b <= Real.1 + -a(n))
            (Real.1 - a(n) = Real.1 + -a(n))
            (Real.1 - b = Real.1 + -b)
            (one_minus_seq(a)(n) = Real.1 - a(n))
            Real.1 - b <= one_minus_seq(a)(n)
        }
        (is_lower_bound(one_minus_seq(a), Real.1 - b) = forall(n: Nat) {
            Real.1 - b <= one_minus_seq(a)(n)
        })
        is_lower_bound(one_minus_seq(a), Real.1 - b)
        is_bounded_below_intro(one_minus_seq(a), Real.1 - b)
        is_bounded_below(one_minus_seq(a))
    }
}

/// The tail supremum of a reflected sequence is the reflection of the tail infimum.
///
/// Proved at the level of the tails, where it needs no limits: reflection exchanges the roles
/// of the two bounds, and each direction is one application of the characterising properties.
theorem tail_sup_one_minus(a: Nat -> Real, n: Nat) {
    is_bounded_above(a) and is_bounded_below(a)
        implies tail_sup(one_minus_seq(a), n) = Real.1 - tail_inf(a, n)
} by {
    if is_bounded_above(a) and is_bounded_below(a) {
        one_minus_bounded_above(a)
        is_bounded_above(one_minus_seq(a))
        forall(k: Nat) {
            if n <= k {
                tail_inf_bounds_terms(a, n, k)
                tail_inf(a, n) <= a(k)
                neg_reverses_lte(tail_inf(a, n), a(k))
                -a(k) <= -tail_inf(a, n)
                (Real.1 + -a(k) <= Real.1 + -tail_inf(a, n))
                (Real.1 - a(k) = Real.1 + -a(k))
                (Real.1 - tail_inf(a, n) = Real.1 + -tail_inf(a, n))
                (one_minus_seq(a)(k) = Real.1 - a(k))
                one_minus_seq(a)(k) <= Real.1 - tail_inf(a, n)
            }
            (n <= k implies one_minus_seq(a)(k) <= Real.1 - tail_inf(a, n))
        }
        tail_sup_is_least(one_minus_seq(a), n, Real.1 - tail_inf(a, n))
        tail_sup(one_minus_seq(a), n) <= Real.1 - tail_inf(a, n)
        forall(k: Nat) {
            if n <= k {
                tail_sup_bounds_terms(one_minus_seq(a), n, k)
                one_minus_seq(a)(k) <= tail_sup(one_minus_seq(a), n)
                (one_minus_seq(a)(k) = Real.1 - a(k))
                (Real.1 - a(k) <= tail_sup(one_minus_seq(a), n))
                one_minus_flip(a(k), tail_sup(one_minus_seq(a), n))
                (Real.1 - tail_sup(one_minus_seq(a), n) <= a(k))
            }
            (n <= k implies Real.1 - tail_sup(one_minus_seq(a), n) <= a(k))
        }
        tail_inf_is_greatest(a, n, Real.1 - tail_sup(one_minus_seq(a), n))
        Real.1 - tail_sup(one_minus_seq(a), n) <= tail_inf(a, n)
        one_minus_flip(tail_sup(one_minus_seq(a), n), tail_inf(a, n))
        (Real.1 - tail_inf(a, n) <= tail_sup(one_minus_seq(a), n))
        lte_antisymm(tail_sup(one_minus_seq(a), n), Real.1 - tail_inf(a, n))
        tail_sup(one_minus_seq(a), n) = Real.1 - tail_inf(a, n)
    }
}

/// The negated tail suprema of a reflected sequence are the tail infima shifted by one.
theorem neg_tail_sup_one_minus(a: Nat -> Real, n: Nat) {
    is_bounded_above(a) and is_bounded_below(a)
        implies neg_tail_sup(one_minus_seq(a))(n)
            = add_seq(neg_tail_sup(neg_seq(a)), constant[Nat, Real](-Real.1))(n)
} by {
    if is_bounded_above(a) and is_bounded_below(a) {
        tail_sup_one_minus(a, n)
        tail_sup(one_minus_seq(a), n) = Real.1 - tail_inf(a, n)
        (neg_tail_sup(one_minus_seq(a))(n) = -tail_sup(one_minus_seq(a), n))
        (-(Real.1 - tail_inf(a, n)) = tail_inf(a, n) + -Real.1)
        neg_tail_sup(one_minus_seq(a))(n) = tail_inf(a, n) + -Real.1
        neg_tail_sup_neg_seq_eq_tail_inf(a, n)
        neg_tail_sup(neg_seq(a))(n) = tail_inf(a, n)
        (constant[Nat, Real](-Real.1)(n) = -Real.1)
        (add_seq(neg_tail_sup(neg_seq(a)), constant[Nat, Real](-Real.1))(n)
            = neg_tail_sup(neg_seq(a))(n) + constant[Nat, Real](-Real.1)(n))
        (add_seq(neg_tail_sup(neg_seq(a)), constant[Nat, Real](-Real.1))(n)
            = tail_inf(a, n) + -Real.1)
        (neg_tail_sup(one_minus_seq(a))(n)
            = add_seq(neg_tail_sup(neg_seq(a)), constant[Nat, Real](-Real.1))(n))
    }
}

/// The limit superior of a reflected sequence is the reflection of the limit inferior.
///
/// The tail suprema of the reflection are the reflected tail infima, so the sequences whose
/// limits the two invariants are differ by a constant, and the limit of a sum splits.
theorem limsup_one_minus(a: Nat -> Real) {
    is_bounded_above(a) and is_bounded_below(a)
        implies limsup(one_minus_seq(a)) = Real.1 - liminf(a)
} by {
    if is_bounded_above(a) and is_bounded_below(a) {
        neg_seq_bounded_above(a)
        is_bounded_above(neg_seq(a))
        neg_seq_bounded_below(a)
        is_bounded_below(neg_seq(a))
        neg_tail_sup_converges(neg_seq(a))
        converges(neg_tail_sup(neg_seq(a)))
        const_converges(-Real.1)
        converges(constant[Nat, Real](-Real.1))
        limit_add_seq(neg_tail_sup(neg_seq(a)), constant[Nat, Real](-Real.1))
        converges_to(add_seq(neg_tail_sup(neg_seq(a)), constant[Nat, Real](-Real.1)),
            limit(neg_tail_sup(neg_seq(a))) + limit(constant[Nat, Real](-Real.1)))
        const_limit(-Real.1)
        limit(constant[Nat, Real](-Real.1)) = -Real.1
        liminf_eq_limit_tail_inf(a)
        liminf(a) = limit(neg_tail_sup(neg_seq(a)))
        converges_to(add_seq(neg_tail_sup(neg_seq(a)), constant[Nat, Real](-Real.1)),
            liminf(a) + -Real.1)
        forall(n: Nat) {
            neg_tail_sup_one_minus(a, n)
            (neg_tail_sup(one_minus_seq(a))(n)
                = add_seq(neg_tail_sup(neg_seq(a)), constant[Nat, Real](-Real.1))(n))
        }
        function_extensionality[Nat, Real](neg_tail_sup(one_minus_seq(a)),
            add_seq(neg_tail_sup(neg_seq(a)), constant[Nat, Real](-Real.1)))
        (neg_tail_sup(one_minus_seq(a))
            = add_seq(neg_tail_sup(neg_seq(a)), constant[Nat, Real](-Real.1)))
        converges_to(neg_tail_sup(one_minus_seq(a)), liminf(a) + -Real.1)
        converges_to_imp_converges(neg_tail_sup(one_minus_seq(a)), liminf(a) + -Real.1)
        converges(neg_tail_sup(one_minus_seq(a)))
        convergent_converges_to_limit(neg_tail_sup(one_minus_seq(a)))
        converges_to(neg_tail_sup(one_minus_seq(a)), limit(neg_tail_sup(one_minus_seq(a))))
        converges_to_unique(neg_tail_sup(one_minus_seq(a)), liminf(a) + -Real.1,
            limit(neg_tail_sup(one_minus_seq(a))))
        liminf(a) + -Real.1 = limit(neg_tail_sup(one_minus_seq(a)))
        (limsup(one_minus_seq(a)) = -limit(neg_tail_sup(one_minus_seq(a))))
        (-(liminf(a) + -Real.1) = Real.1 - liminf(a))
        limsup(one_minus_seq(a)) = Real.1 - liminf(a)
    }
}
