from nat import Nat
from real import Real, lte_trans, add_seq, converges, converges_to, limit, seq_lte,
    seq_lte_preserves_limit, limit_add_seq, converges_to_imp_converges,
    converges_to_unique, convergent_converges_to_limit, is_upper_bound, is_lower_bound,
    lte_add_right, lte_add_left
from analysis.real.real_tail_supremum import tail_sup, tail_sup_bounds_terms, tail_sup_is_least,
    is_bounded_above, is_bounded_above_intro, is_bounded_above_witness
from analysis.real.real_limsup import limsup, neg_reverses_lte, is_bounded_below, is_bounded_below_intro,
    is_bounded_below_witness, neg_tail_sup, neg_tail_sup_converges

/// A sum of sequences bounded above is bounded above.
theorem add_seq_bounded_above(a: Nat -> Real, b: Nat -> Real) {
    is_bounded_above(a) and is_bounded_above(b) implies is_bounded_above(add_seq(a, b))
} by {
    if is_bounded_above(a) and is_bounded_above(b) {
        is_bounded_above_witness(a)
        exists(c: Real) { is_upper_bound(a, c) }
        let (u: Real) satisfy {
            is_upper_bound(a, u)
        }
        is_bounded_above_witness(b)
        exists(c: Real) { is_upper_bound(b, c) }
        let (v: Real) satisfy {
            is_upper_bound(b, v)
        }
        (is_upper_bound(a, u) = forall(k: Nat) { a(k) <= u })
        (is_upper_bound(b, v) = forall(k: Nat) { b(k) <= v })
        forall(n: Nat) {
            a(n) <= u
            b(n) <= v
            (a(n) + b(n) <= u + v)
            (add_seq(a, b)(n) = a(n) + b(n))
            add_seq(a, b)(n) <= u + v
        }
        (is_upper_bound(add_seq(a, b), u + v) = forall(n: Nat) {
            add_seq(a, b)(n) <= u + v
        })
        is_upper_bound(add_seq(a, b), u + v)
        is_bounded_above_intro(add_seq(a, b), u + v)
        is_bounded_above(add_seq(a, b))
    }
}

/// A sum of sequences bounded below is bounded below.
theorem add_seq_bounded_below(a: Nat -> Real, b: Nat -> Real) {
    is_bounded_below(a) and is_bounded_below(b) implies is_bounded_below(add_seq(a, b))
} by {
    if is_bounded_below(a) and is_bounded_below(b) {
        is_bounded_below_witness(a)
        exists(c: Real) { is_lower_bound(a, c) }
        let (u: Real) satisfy {
            is_lower_bound(a, u)
        }
        is_bounded_below_witness(b)
        exists(c: Real) { is_lower_bound(b, c) }
        let (v: Real) satisfy {
            is_lower_bound(b, v)
        }
        (is_lower_bound(a, u) = forall(k: Nat) { u <= a(k) })
        (is_lower_bound(b, v) = forall(k: Nat) { v <= b(k) })
        forall(n: Nat) {
            u <= a(n)
            v <= b(n)
            (u + v <= a(n) + b(n))
            (add_seq(a, b)(n) = a(n) + b(n))
            u + v <= add_seq(a, b)(n)
        }
        (is_lower_bound(add_seq(a, b), u + v) = forall(n: Nat) {
            u + v <= add_seq(a, b)(n)
        })
        is_lower_bound(add_seq(a, b), u + v)
        is_bounded_below_intro(add_seq(a, b), u + v)
        is_bounded_below(add_seq(a, b))
    }
}

/// The tail supremum of a sum is at most the sum of the tail suprema.
///
/// At the level of the tails, where it is one application of each characterising property.
theorem tail_sup_add(a: Nat -> Real, b: Nat -> Real, n: Nat) {
    is_bounded_above(a) and is_bounded_above(b)
        implies tail_sup(add_seq(a, b), n) <= tail_sup(a, n) + tail_sup(b, n)
} by {
    if is_bounded_above(a) and is_bounded_above(b) {
        add_seq_bounded_above(a, b)
        is_bounded_above(add_seq(a, b))
        forall(k: Nat) {
            if n <= k {
                tail_sup_bounds_terms(a, n, k)
                a(k) <= tail_sup(a, n)
                tail_sup_bounds_terms(b, n, k)
                b(k) <= tail_sup(b, n)
                (a(k) + b(k) <= tail_sup(a, n) + tail_sup(b, n))
                (add_seq(a, b)(k) = a(k) + b(k))
                add_seq(a, b)(k) <= tail_sup(a, n) + tail_sup(b, n)
            }
            (n <= k implies add_seq(a, b)(k) <= tail_sup(a, n) + tail_sup(b, n))
        }
        tail_sup_is_least(add_seq(a, b), n, tail_sup(a, n) + tail_sup(b, n))
        tail_sup(add_seq(a, b), n) <= tail_sup(a, n) + tail_sup(b, n)
    }
}

/// The limit superior of a sum is at most the sum of the limit superiors.
///
/// Negating turns the tail bound around, and the limit of a sum splits, so the comparison passes
/// to the limits.
theorem limsup_add(a: Nat -> Real, b: Nat -> Real) {
    is_bounded_above(a) and is_bounded_below(a) and is_bounded_above(b)
        and is_bounded_below(b)
        implies limsup(add_seq(a, b)) <= limsup(a) + limsup(b)
} by {
    if is_bounded_above(a) and is_bounded_below(a) and is_bounded_above(b)
        and is_bounded_below(b) {
        add_seq_bounded_above(a, b)
        is_bounded_above(add_seq(a, b))
        add_seq_bounded_below(a, b)
        is_bounded_below(add_seq(a, b))
        neg_tail_sup_converges(a)
        converges(neg_tail_sup(a))
        neg_tail_sup_converges(b)
        converges(neg_tail_sup(b))
        neg_tail_sup_converges(add_seq(a, b))
        converges(neg_tail_sup(add_seq(a, b)))
        limit_add_seq(neg_tail_sup(a), neg_tail_sup(b))
        converges_to(add_seq(neg_tail_sup(a), neg_tail_sup(b)),
            limit(neg_tail_sup(a)) + limit(neg_tail_sup(b)))
        converges_to_imp_converges(add_seq(neg_tail_sup(a), neg_tail_sup(b)),
            limit(neg_tail_sup(a)) + limit(neg_tail_sup(b)))
        converges(add_seq(neg_tail_sup(a), neg_tail_sup(b)))
        convergent_converges_to_limit(add_seq(neg_tail_sup(a), neg_tail_sup(b)))
        converges_to(add_seq(neg_tail_sup(a), neg_tail_sup(b)),
            limit(add_seq(neg_tail_sup(a), neg_tail_sup(b))))
        converges_to_unique(add_seq(neg_tail_sup(a), neg_tail_sup(b)),
            limit(neg_tail_sup(a)) + limit(neg_tail_sup(b)),
            limit(add_seq(neg_tail_sup(a), neg_tail_sup(b))))
        (limit(neg_tail_sup(a)) + limit(neg_tail_sup(b))
            = limit(add_seq(neg_tail_sup(a), neg_tail_sup(b))))
        forall(n: Nat) {
            tail_sup_add(a, b, n)
            tail_sup(add_seq(a, b), n) <= tail_sup(a, n) + tail_sup(b, n)
            neg_reverses_lte(tail_sup(add_seq(a, b), n),
                tail_sup(a, n) + tail_sup(b, n))
            (-(tail_sup(a, n) + tail_sup(b, n)) <= -tail_sup(add_seq(a, b), n))
            (-(tail_sup(a, n) + tail_sup(b, n)) = -tail_sup(a, n) + -tail_sup(b, n))
            (neg_tail_sup(a)(n) = -tail_sup(a, n))
            (neg_tail_sup(b)(n) = -tail_sup(b, n))
            (add_seq(neg_tail_sup(a), neg_tail_sup(b))(n)
                = neg_tail_sup(a)(n) + neg_tail_sup(b)(n))
            (neg_tail_sup(add_seq(a, b))(n) = -tail_sup(add_seq(a, b), n))
            (add_seq(neg_tail_sup(a), neg_tail_sup(b))(n)
                <= neg_tail_sup(add_seq(a, b))(n))
        }
        (seq_lte(add_seq(neg_tail_sup(a), neg_tail_sup(b)),
            neg_tail_sup(add_seq(a, b))) = forall(n: Nat) {
            add_seq(neg_tail_sup(a), neg_tail_sup(b))(n)
                <= neg_tail_sup(add_seq(a, b))(n)
        })
        seq_lte(add_seq(neg_tail_sup(a), neg_tail_sup(b)), neg_tail_sup(add_seq(a, b)))
        seq_lte_preserves_limit(add_seq(neg_tail_sup(a), neg_tail_sup(b)),
            neg_tail_sup(add_seq(a, b)))
        (limit(add_seq(neg_tail_sup(a), neg_tail_sup(b)))
            <= limit(neg_tail_sup(add_seq(a, b))))
        (limit(neg_tail_sup(a)) + limit(neg_tail_sup(b))
            <= limit(neg_tail_sup(add_seq(a, b))))
        neg_reverses_lte(limit(neg_tail_sup(a)) + limit(neg_tail_sup(b)),
            limit(neg_tail_sup(add_seq(a, b))))
        (-limit(neg_tail_sup(add_seq(a, b)))
            <= -(limit(neg_tail_sup(a)) + limit(neg_tail_sup(b))))
        ((-(limit(neg_tail_sup(a)) + limit(neg_tail_sup(b))))
            = -limit(neg_tail_sup(a)) + -limit(neg_tail_sup(b)))
        (limsup(add_seq(a, b)) = -limit(neg_tail_sup(add_seq(a, b))))
        (limsup(a) = -limit(neg_tail_sup(a)))
        (limsup(b) = -limit(neg_tail_sup(b)))
        limsup(add_seq(a, b)) <= limsup(a) + limsup(b)
    }
}
