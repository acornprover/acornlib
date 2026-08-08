from nat import Nat
from rat import Rat
from real import Real, converges_to, tail_bound, tail_bound_implies_is_close, close_imp_bounds,
    add_rat_eps_between, from_rat_pos, lte_antisymm, lte_trans, eventual_ub, ub_imp_limit_lte,
    limit, converges, lt_lte_trans, lte_add_right
from analysis.real.real_tail_supremum import tail_sup, tail_sup_bounds_terms, tail_sup_is_least,
    is_bounded_above
from analysis.real.real_limsup import limsup, limsup_le_tail_sup, neg_reverses_lte, is_bounded_below,
    neg_tail_sup, neg_tail_sup_converges

/// A real below every positive shift of another is below it.
///
/// The standard way to close an epsilon argument. `src/real/` supplies the ingredient — between
/// any two distinct reals there is a positive rational gap — but not this statement.
theorem lte_of_lte_add_eps(x: Real, y: Real) {
    (forall(eps: Real) { eps.is_positive implies x <= y + eps }) implies x <= y
} by {
    if forall(eps: Real) { eps.is_positive implies x <= y + eps } {
        if y < x {
            add_rat_eps_between(y, x)
            exists(e: Rat) {
                e.is_positive and y + Real.from_rat(e) < x
            }
            let (e: Rat) satisfy {
                e.is_positive and y + Real.from_rat(e) < x
            }
            from_rat_pos(e)
            Real.from_rat(e).is_positive
            (Real.from_rat(e).is_positive implies x <= y + Real.from_rat(e))
            x <= y + Real.from_rat(e)
            y + Real.from_rat(e) < x
            x < x
            false
        }
        not (y < x)
        x <= y
    }
}

/// The limit superior of a convergent sequence is at most its limit.
///
/// Past the threshold every term is below the limit plus a size, so the tail supremum there is
/// too, and the limit superior is below that tail supremum.
theorem limsup_le_limit(a: Nat -> Real, l: Real) {
    is_bounded_above(a) and is_bounded_below(a) and converges_to(a, l)
        implies limsup(a) <= l
} by {
    if is_bounded_above(a) and is_bounded_below(a) and converges_to(a, l) {
        forall(eps: Real) {
            if eps.is_positive {
                (converges_to(a, l) = forall(d: Real) {
                    d.is_positive implies exists(m: Nat) { tail_bound(a, l, m, d) }
                })
                exists(m: Nat) { tail_bound(a, l, m, eps) }
                let (n: Nat) satisfy {
                    tail_bound(a, l, n, eps)
                }
                forall(k: Nat) {
                    if n <= k {
                        tail_bound_implies_is_close(a, l, n, eps, k)
                        a(k).is_close(l, eps)
                        close_imp_bounds(a(k), l, eps)
                        a(k) < l + eps
                        a(k) <= l + eps
                    }
                    (n <= k implies a(k) <= l + eps)
                }
                tail_sup_is_least(a, n, l + eps)
                tail_sup(a, n) <= l + eps
                limsup_le_tail_sup(a, n)
                limsup(a) <= tail_sup(a, n)
                lte_trans(limsup(a), tail_sup(a, n), l + eps)
                limsup(a) <= l + eps
            }
            (eps.is_positive implies limsup(a) <= l + eps)
        }
        lte_of_lte_add_eps(limsup(a), l)
        limsup(a) <= l
    }
}

/// The limit of a convergent sequence is at most its limit superior.
///
/// Past the threshold every term is above the limit minus a size, so every tail supremum is too,
/// since each tail contains a term past the threshold.
theorem limit_le_limsup(a: Nat -> Real, l: Real) {
    is_bounded_above(a) and is_bounded_below(a) and converges_to(a, l)
        implies l <= limsup(a)
} by {
    if is_bounded_above(a) and is_bounded_below(a) and converges_to(a, l) {
        neg_tail_sup_converges(a)
        converges(neg_tail_sup(a))
        forall(eps: Real) {
            if eps.is_positive {
                (converges_to(a, l) = forall(d: Real) {
                    d.is_positive implies exists(m: Nat) { tail_bound(a, l, m, d) }
                })
                exists(m: Nat) { tail_bound(a, l, m, eps) }
                let (n: Nat) satisfy {
                    tail_bound(a, l, n, eps)
                }
                forall(j: Nat) {
                    if Nat.0 <= j {
                        n <= j + n
                        tail_bound_implies_is_close(a, l, n, eps, j + n)
                        a(j + n).is_close(l, eps)
                        close_imp_bounds(a(j + n), l, eps)
                        l - eps < a(j + n)
                        j <= j + n
                        tail_sup_bounds_terms(a, j, j + n)
                        a(j + n) <= tail_sup(a, j)
                        lt_lte_trans(l - eps, a(j + n), tail_sup(a, j))
                        l - eps < tail_sup(a, j)
                        l - eps <= tail_sup(a, j)
                        neg_reverses_lte(l - eps, tail_sup(a, j))
                        -tail_sup(a, j) <= -(l - eps)
                        (neg_tail_sup(a)(j) = -tail_sup(a, j))
                        neg_tail_sup(a)(j) <= -(l - eps)
                    }
                    (Nat.0 <= j implies neg_tail_sup(a)(j) <= -(l - eps))
                }
                (eventual_ub(neg_tail_sup(a), -(l - eps)) = exists(m: Nat) {
                    forall(i: Nat) {
                        m <= i implies neg_tail_sup(a)(i) <= -(l - eps)
                    }
                })
                exists(m: Nat) {
                    forall(i: Nat) {
                        m <= i implies neg_tail_sup(a)(i) <= -(l - eps)
                    }
                }
                eventual_ub(neg_tail_sup(a), -(l - eps))
                ub_imp_limit_lte(neg_tail_sup(a), -(l - eps))
                limit(neg_tail_sup(a)) <= -(l - eps)
                neg_reverses_lte(limit(neg_tail_sup(a)), -(l - eps))
                (-(-(l - eps)) <= -limit(neg_tail_sup(a)))
                ((-(-(l - eps))) = l - eps)
                (limsup(a) = -limit(neg_tail_sup(a)))
                l - eps <= limsup(a)
                lte_add_right(l - eps, limsup(a), eps)
                (l - eps) + eps <= limsup(a) + eps
                ((l - eps) + eps = l)
                (l <= limsup(a) + eps)
            }
            (eps.is_positive implies l <= limsup(a) + eps)
        }
        lte_of_lte_add_eps(l, limsup(a))
        l <= limsup(a)
    }
}

/// The limit superior of a convergent sequence is its limit.
///
/// This is what makes the upper density agree with the natural density wherever the latter
/// exists.
theorem limsup_eq_limit(a: Nat -> Real, l: Real) {
    is_bounded_above(a) and is_bounded_below(a) and converges_to(a, l)
        implies limsup(a) = l
} by {
    if is_bounded_above(a) and is_bounded_below(a) and converges_to(a, l) {
        limsup_le_limit(a, l)
        limsup(a) <= l
        limit_le_limsup(a, l)
        l <= limsup(a)
        lte_antisymm(limsup(a), l)
        limsup(a) = l
    }
}
