from nat import Nat, lte_trans
from data.basic.set import Set
from real import Real, is_upper_bound, is_nonempty, is_set_upper_bound, has_upper_bound,
    is_set_supremum, completeness, supremum_subset_le

/// True if `x` is a value the sequence takes at or after index `n`.
define tail_value(a: Nat -> Real, n: Nat, x: Real) -> Bool {
    exists(k: Nat) {
        n <= k and a(k) = x
    }
}

/// The set of values a sequence takes at or after index `n`.
///
/// The tail of a sequence as a set rather than as a sequence, which is the shape the completeness
/// property of the reals is stated for.
define tail_set(a: Nat -> Real, n: Nat) -> Set[Real] {
    Set[Real].new(tail_value(a, n))
}

/// Membership in the tail set is being a value from the index onward.
theorem tail_set_contains_eq(a: Nat -> Real, n: Nat, x: Real) {
    tail_set(a, n).contains(x) = tail_value(a, n, x)
} by {
    (tail_set(a, n).contains = tail_value(a, n))
    tail_set(a, n).contains(x) = tail_value(a, n, x)
}

/// Every value from the index onward lies in the tail set.
theorem tail_set_contains(a: Nat -> Real, n: Nat, k: Nat) {
    n <= k implies tail_set(a, n).contains(a(k))
} by {
    if n <= k {
        exists(j: Nat) { n <= j and a(j) = a(k) }
        (tail_value(a, n, a(k)) = exists(j: Nat) { n <= j and a(j) = a(k) })
        tail_value(a, n, a(k))
        tail_set_contains_eq(a, n, a(k))
        tail_set(a, n).contains(a(k))
    }
}

/// A member of the tail set is a value from the index onward.
theorem tail_set_witness(a: Nat -> Real, n: Nat, x: Real) {
    tail_set(a, n).contains(x) implies exists(k: Nat) { n <= k and a(k) = x }
} by {
    if tail_set(a, n).contains(x) {
        tail_set_contains_eq(a, n, x)
        tail_value(a, n, x)
        (tail_value(a, n, x) = exists(k: Nat) { n <= k and a(k) = x })
        exists(k: Nat) { n <= k and a(k) = x }
    }
}

/// A tail set is never empty.
theorem tail_set_nonempty(a: Nat -> Real, n: Nat) {
    is_nonempty(tail_set(a, n))
} by {
    n <= n
    tail_set_contains(a, n, n)
    tail_set(a, n).contains(a(n))
    exists(x: Real) { tail_set(a, n).contains(x) }
    (is_nonempty(tail_set(a, n)) = exists(x: Real) { tail_set(a, n).contains(x) })
    is_nonempty(tail_set(a, n))
}

/// A bound on the sequence bounds every tail set.
theorem tail_set_upper_bound(a: Nat -> Real, b: Real, n: Nat) {
    is_upper_bound(a, b) implies is_set_upper_bound(tail_set(a, n), b)
} by {
    if is_upper_bound(a, b) {
        (is_upper_bound(a, b) = forall(k: Nat) { a(k) <= b })
        forall(x: Real) {
            if tail_set(a, n).contains(x) {
                tail_set_witness(a, n, x)
                exists(k: Nat) { n <= k and a(k) = x }
                let (k: Nat) satisfy {
                    n <= k and a(k) = x
                }
                a(k) <= b
                x <= b
            }
            (tail_set(a, n).contains(x) implies x <= b)
        }
        (is_set_upper_bound(tail_set(a, n), b) = forall(y: Real) {
            tail_set(a, n).contains(y) implies y <= b
        })
        is_set_upper_bound(tail_set(a, n), b)
    }
}

/// True if some real bounds every term of the sequence.
define is_bounded_above(a: Nat -> Real) -> Bool {
    exists(b: Real) {
        is_upper_bound(a, b)
    }
}

/// A bound witnesses boundedness above.
theorem is_bounded_above_intro(a: Nat -> Real, b: Real) {
    is_upper_bound(a, b) implies is_bounded_above(a)
} by {
    if is_upper_bound(a, b) {
        exists(c: Real) { is_upper_bound(a, c) }
        (is_bounded_above(a) = exists(c: Real) { is_upper_bound(a, c) })
        is_bounded_above(a)
    }
}

/// Boundedness above yields a bound.
theorem is_bounded_above_witness(a: Nat -> Real) {
    is_bounded_above(a) implies exists(b: Real) { is_upper_bound(a, b) }
} by {
    if is_bounded_above(a) {
        (is_bounded_above(a) = exists(c: Real) { is_upper_bound(a, c) })
        exists(c: Real) { is_upper_bound(a, c) }
    }
}

/// The least bound on the values a sequence takes from index `n` onward.
///
/// Defined by completeness, which applies because the tail is nonempty and inherits any bound on
/// the whole sequence. The defining property is stated only for a sequence bounded above, since
/// without a bound there is no supremum; the function itself is total.
let tail_sup(a: Nat -> Real, n: Nat) -> result: Real satisfy {
    is_bounded_above(a) implies is_set_supremum(tail_set(a, n), result)
} by {
    if is_bounded_above(a) {
        is_bounded_above_witness(a)
        exists(b: Real) { is_upper_bound(a, b) }
        let (b: Real) satisfy {
            is_upper_bound(a, b)
        }
        tail_set_upper_bound(a, b, n)
        is_set_upper_bound(tail_set(a, n), b)
        exists(c: Real) { is_set_upper_bound(tail_set(a, n), c) }
        (has_upper_bound(tail_set(a, n)) = exists(c: Real) {
            is_set_upper_bound(tail_set(a, n), c)
        })
        has_upper_bound(tail_set(a, n))
        tail_set_nonempty(a, n)
        is_nonempty(tail_set(a, n))
        completeness(tail_set(a, n))
        exists(sup: Real) { is_set_supremum(tail_set(a, n), sup) }
    }
}

/// The tail supremum is a supremum of the tail set.
theorem tail_sup_is_supremum(a: Nat -> Real, n: Nat) {
    is_bounded_above(a) implies is_set_supremum(tail_set(a, n), tail_sup(a, n))
} by {
    if is_bounded_above(a) {
        is_set_supremum(tail_set(a, n), tail_sup(a, n))
    }
}

/// No term from the index onward exceeds the tail supremum.
theorem tail_sup_bounds_terms(a: Nat -> Real, n: Nat, k: Nat) {
    is_bounded_above(a) and n <= k implies a(k) <= tail_sup(a, n)
} by {
    if is_bounded_above(a) and n <= k {
        tail_sup_is_supremum(a, n)
        is_set_supremum(tail_set(a, n), tail_sup(a, n))
        (is_set_supremum(tail_set(a, n), tail_sup(a, n))
            = (is_set_upper_bound(tail_set(a, n), tail_sup(a, n)) and forall(b: Real) {
                is_set_upper_bound(tail_set(a, n), b) implies tail_sup(a, n) <= b
            }))
        is_set_upper_bound(tail_set(a, n), tail_sup(a, n))
        (is_set_upper_bound(tail_set(a, n), tail_sup(a, n)) = forall(x: Real) {
            tail_set(a, n).contains(x) implies x <= tail_sup(a, n)
        })
        tail_set_contains(a, n, k)
        tail_set(a, n).contains(a(k))
        a(k) <= tail_sup(a, n)
    }
}

/// Nothing below the tail supremum bounds the tail.
theorem tail_sup_is_least(a: Nat -> Real, n: Nat, b: Real) {
    is_bounded_above(a) and (forall(k: Nat) { n <= k implies a(k) <= b })
        implies tail_sup(a, n) <= b
} by {
    if is_bounded_above(a) and forall(k: Nat) { n <= k implies a(k) <= b } {
        forall(x: Real) {
            if tail_set(a, n).contains(x) {
                tail_set_witness(a, n, x)
                exists(k: Nat) { n <= k and a(k) = x }
                let (k: Nat) satisfy {
                    n <= k and a(k) = x
                }
                (n <= k implies a(k) <= b)
                a(k) <= b
                x <= b
            }
            (tail_set(a, n).contains(x) implies x <= b)
        }
        (is_set_upper_bound(tail_set(a, n), b) = forall(y: Real) {
            tail_set(a, n).contains(y) implies y <= b
        })
        is_set_upper_bound(tail_set(a, n), b)
        tail_sup_is_supremum(a, n)
        is_set_supremum(tail_set(a, n), tail_sup(a, n))
        (is_set_supremum(tail_set(a, n), tail_sup(a, n))
            = (is_set_upper_bound(tail_set(a, n), tail_sup(a, n)) and forall(c: Real) {
                is_set_upper_bound(tail_set(a, n), c) implies tail_sup(a, n) <= c
            }))
        forall(c: Real) {
            is_set_upper_bound(tail_set(a, n), c) implies tail_sup(a, n) <= c
        }
        (is_set_upper_bound(tail_set(a, n), b) implies tail_sup(a, n) <= b)
        tail_sup(a, n) <= b
    }
}

/// Later tails sit inside earlier ones.
theorem tail_set_subset(a: Nat -> Real, m: Nat, n: Nat) {
    m <= n implies tail_set(a, n).subset(tail_set(a, m))
} by {
    if m <= n {
        forall(x: Real) {
            if tail_set(a, n).contains(x) {
                tail_set_witness(a, n, x)
                exists(k: Nat) { n <= k and a(k) = x }
                let (k: Nat) satisfy {
                    n <= k and a(k) = x
                }
                lte_trans(m, n, k)
                m <= k
                tail_set_contains(a, m, k)
                tail_set(a, m).contains(a(k))
                tail_set(a, m).contains(x)
            }
            (tail_set(a, n).contains(x) implies tail_set(a, m).contains(x))
        }
        (tail_set(a, n).subset(tail_set(a, m)) = forall(y: Real) {
            tail_set(a, n).contains(y) implies tail_set(a, m).contains(y)
        })
        tail_set(a, n).subset(tail_set(a, m))
    }
}

/// The tail supremum does not increase as the index grows.
///
/// Discarding early terms can only remove candidates, so the sequence of tail suprema is
/// non-increasing. This is what makes the limit superior an infimum of a monotone sequence.
theorem tail_sup_antitone(a: Nat -> Real, m: Nat, n: Nat) {
    is_bounded_above(a) and m <= n implies tail_sup(a, n) <= tail_sup(a, m)
} by {
    if is_bounded_above(a) and m <= n {
        tail_set_subset(a, m, n)
        tail_set(a, n).subset(tail_set(a, m))
        tail_sup_is_supremum(a, n)
        is_set_supremum(tail_set(a, n), tail_sup(a, n))
        tail_sup_is_supremum(a, m)
        is_set_supremum(tail_set(a, m), tail_sup(a, m))
        supremum_subset_le(tail_set(a, n), tail_set(a, m), tail_sup(a, n), tail_sup(a, m))
        tail_sup(a, n) <= tail_sup(a, m)
    }
}
