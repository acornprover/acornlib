from nat import Nat
from int import Int, abs, abs_zero_imp_zero, pos_is_not_neg, lte_from_nat,
    lte_add_left, zero_lt_pos, sub_nonnegative_of_lte, add_sub_lt_right_of_lt
from rat import Rat, lte_from_int, mul_int_eq_int_mul, from_int_num
from real import Real, floor, from_rat_maintains_lte, mul_from_rat,
    lt_some_int_cancel, mul_le_mul_pos_left, from_rat_pos, lte_trans, lt_lte_trans, lte_lt_trans

numerals Int

/// The embedding of the integers into the reals preserves order.
theorem real_from_int_lte(m: Int, n: Int) {
    m <= n implies Real.from_int(m) <= Real.from_int(n)
} by {
    if m <= n {
        lte_from_int(m, n)
        (Rat.from_int(m) <= Rat.from_int(n))
        from_rat_maintains_lte(Rat.from_int(m), Rat.from_int(n))
        (Real.from_rat(Rat.from_int(m)) <= Real.from_rat(Rat.from_int(n)))
        (Real.from_int(m) = Real.from_rat(Rat.from_int(m)))
        (Real.from_int(n) = Real.from_rat(Rat.from_int(n)))
        Real.from_int(m) <= Real.from_int(n)
    }
}

/// The embedding of the integers into the reals preserves products.
theorem real_from_int_mul(m: Int, n: Int) {
    Real.from_int(m) * Real.from_int(n) = Real.from_int(m * n)
} by {
    (Real.from_int(m) = Real.from_rat(Rat.from_int(m)))
    (Real.from_int(n) = Real.from_rat(Rat.from_int(n)))
    mul_from_rat(Rat.from_int(m), Rat.from_int(n))
    (Real.from_rat(Rat.from_int(m)) * Real.from_rat(Rat.from_int(n))
        = Real.from_rat(Rat.from_int(m) * Rat.from_int(n)))
    mul_int_eq_int_mul(m, n)
    (Rat.from_int(m) * Rat.from_int(n) = Rat.from_int(m * n))
    (Real.from_int(m * n) = Real.from_rat(Rat.from_int(m * n)))
    Real.from_int(m) * Real.from_int(n) = Real.from_int(m * n)
}

/// The integers are discrete: strictly above means at least one more.
///
/// `src/int/` states monotonicity of the order in both strengths but nothing that turns a strict
/// inequality into a bound by the successor, which every floor argument needs.
theorem int_lt_imp_add_one_lte(a: Int, b: Int) {
    a < b implies a + Int.1 <= b
} by {
    if a < b {
        a <= b
        a != b
        ((b - a).is_positive or a = b)
        (b - a).is_positive
        pos_is_not_neg(b - a)
        not (b - a).is_negative
        (b - a = Int.from_nat(abs(b - a)))
        if abs(b - a) = Nat.0 {
            abs_zero_imp_zero(b - a)
            (b - a = Int.0)
            a = b
            false
        }
        abs(b - a) != Nat.0
        Nat.0 < abs(b - a)
        (Nat.0.suc = Nat.1)
        Nat.1 <= abs(b - a)
        lte_from_nat(Nat.1, abs(b - a))
        (Int.from_nat(Nat.1) <= Int.from_nat(abs(b - a)))
        (Int.from_nat(Nat.1) = Int.1)
        Int.1 <= b - a
        lte_add_left(a, Int.1, b - a)
        (a + Int.1 <= a + (b - a))
        (a + (b - a) = b)
        a + Int.1 <= b
    }
}

/// The floor is the greatest integer below a real.
///
/// Its defining property gives that it lies below and that one more does not; this is the other
/// half, that nothing below the real exceeds it.
theorem floor_is_greatest(x: Real, n: Int) {
    Real.from_int(n) <= x implies n <= floor(x)
} by {
    if Real.from_int(n) <= x {
        if floor(x) < n {
            int_lt_imp_add_one_lte(floor(x), n)
            floor(x) + Int.1 <= n
            real_from_int_lte(floor(x) + Int.1, n)
            (Real.from_int(floor(x) + Int.1) <= Real.from_int(n))
            (x < Real.from_int(floor(x) + Int.1))
            lt_lte_trans(x, Real.from_int(floor(x) + Int.1), Real.from_int(n))
            x < Real.from_int(n)
            false
        }
        n <= floor(x)
    }
}

/// Scaling by a positive integer moves the floor by that factor, up to a remainder below it.
///
/// The whole content of a digit expansion: the floor of the scaled value is the scaled floor plus
/// something in the range of digits.
theorem floor_scale_bounds(b: Int, y: Real) {
    b.is_positive implies b * floor(y) <= floor(Real.from_int(b) * y)
        and floor(Real.from_int(b) * y) < b * floor(y) + b
} by {
    if b.is_positive {
        from_int_num(b)
        (Rat.from_int(b).num = b)
        (Rat.from_int(b).is_positive = Rat.from_int(b).num.is_positive)
        Rat.from_int(b).is_positive
        from_rat_pos(Rat.from_int(b))
        Real.from_rat(Rat.from_int(b)).is_positive
        (Real.from_int(b) = Real.from_rat(Rat.from_int(b)))
        Real.from_int(b).is_positive
        Real.from_int(b) > Real.0
        (Real.from_int(floor(y)) <= y)
        mul_le_mul_pos_left(Real.from_int(floor(y)), y, Real.from_int(b))
        (Real.from_int(b) * Real.from_int(floor(y)) <= Real.from_int(b) * y)
        real_from_int_mul(b, floor(y))
        (Real.from_int(b) * Real.from_int(floor(y)) = Real.from_int(b * floor(y)))
        (Real.from_int(b * floor(y)) <= Real.from_int(b) * y)
        floor_is_greatest(Real.from_int(b) * y, b * floor(y))
        b * floor(y) <= floor(Real.from_int(b) * y)
        (y < Real.from_int(floor(y) + Int.1))
        mul_le_mul_pos_left(y, Real.from_int(floor(y) + Int.1), Real.from_int(b))
        (Real.from_int(b) * y < Real.from_int(b) * Real.from_int(floor(y) + Int.1))
        real_from_int_mul(b, floor(y) + Int.1)
        (Real.from_int(b) * Real.from_int(floor(y) + Int.1)
            = Real.from_int(b * (floor(y) + Int.1)))
        (b * (floor(y) + Int.1) = b * floor(y) + b)
        (Real.from_int(b) * y < Real.from_int(b * floor(y) + b))
        (Real.from_int(floor(Real.from_int(b) * y)) <= Real.from_int(b) * y)
        lte_lt_trans(Real.from_int(floor(Real.from_int(b) * y)), Real.from_int(b) * y,
            Real.from_int(b * floor(y) + b))
        (Real.from_int(floor(Real.from_int(b) * y)) < Real.from_int(b * floor(y) + b))
        lt_some_int_cancel(floor(Real.from_int(b) * y), b * floor(y) + b)
        floor(Real.from_int(b) * y) < b * floor(y) + b
        (b * floor(y) <= floor(Real.from_int(b) * y)
            and floor(Real.from_int(b) * y) < b * floor(y) + b)
    }
}

/// The `k`th digit of a real number in base `b`.
///
/// The usual formula: the scaled floor at one more power, less `b` times the scaled floor at this
/// one. The base is taken as a positive integer rather than a natural, since the floor already
/// lands in the integers and no embedding is then needed.
define real_digit(b: Int, x: Real, k: Nat) -> Int {
    floor(Real.from_int(b).pow(k.suc) * x) - b * floor(Real.from_int(b).pow(k) * x)
}

/// The scaled value at one more power is the base times the scaled value here.
theorem real_scaled_suc(b: Int, x: Real, k: Nat) {
    Real.from_int(b).pow(k.suc) * x = Real.from_int(b) * (Real.from_int(b).pow(k) * x)
} by {
    (Real.from_int(b).pow(k.suc) = Real.from_int(b).pow(k) * Real.from_int(b))
    ((Real.from_int(b).pow(k) * Real.from_int(b)) * x
        = Real.from_int(b) * (Real.from_int(b).pow(k) * x))
    Real.from_int(b).pow(k.suc) * x = Real.from_int(b) * (Real.from_int(b).pow(k) * x)
}

/// Every digit lies between zero and the base.
///
/// Exactly the scaling bound, read at the value already scaled by `k` powers: the floor of the
/// scaled value is the scaled floor plus a remainder below the base, and that remainder is the
/// digit.
theorem real_digit_bounds(b: Int, x: Real, k: Nat) {
    b.is_positive implies Int.0 <= real_digit(b, x, k) and real_digit(b, x, k) < b
} by {
    if b.is_positive {
        floor_scale_bounds(b, Real.from_int(b).pow(k) * x)
        (b * floor(Real.from_int(b).pow(k) * x)
            <= floor(Real.from_int(b) * (Real.from_int(b).pow(k) * x))
            and floor(Real.from_int(b) * (Real.from_int(b).pow(k) * x))
                < b * floor(Real.from_int(b).pow(k) * x) + b)
        real_scaled_suc(b, x, k)
        (Real.from_int(b).pow(k.suc) * x
            = Real.from_int(b) * (Real.from_int(b).pow(k) * x))
        (b * floor(Real.from_int(b).pow(k) * x)
            <= floor(Real.from_int(b).pow(k.suc) * x))
        (floor(Real.from_int(b).pow(k.suc) * x)
            < b * floor(Real.from_int(b).pow(k) * x) + b)
        (real_digit(b, x, k) = floor(Real.from_int(b).pow(k.suc) * x)
            - b * floor(Real.from_int(b).pow(k) * x))
        sub_nonnegative_of_lte(b * floor(Real.from_int(b).pow(k) * x),
            floor(Real.from_int(b).pow(k.suc) * x))
        (Int.0 <= real_digit(b, x, k))
        add_sub_lt_right_of_lt(floor(Real.from_int(b).pow(k.suc) * x),
            b * floor(Real.from_int(b).pow(k) * x) + b, b)
        (floor(Real.from_int(b).pow(k.suc) * x)
            + b - (b * floor(Real.from_int(b).pow(k) * x) + b) < b)
        (floor(Real.from_int(b).pow(k.suc) * x)
            + b - (b * floor(Real.from_int(b).pow(k) * x) + b)
            = floor(Real.from_int(b).pow(k.suc) * x)
                - b * floor(Real.from_int(b).pow(k) * x))
        real_digit(b, x, k) < b
        (Int.0 <= real_digit(b, x, k) and real_digit(b, x, k) < b)
    }
}
