from nat import Nat, from_nat
from real import Real, div_le_of_mul_le, mul_div_cancel

numerals Real

/// Embedding the naturals into the reals is nonnegative.
theorem from_nat_real_nonnegative(n: Nat) {
    Real.0 <= from_nat[Real](n)
} by {
    define q(x: Nat) -> Bool {
        Real.0 <= from_nat[Real](x)
    }
    from_nat[Real](Nat.0) = Real.0
    Real.0 <= Real.0
    q(Nat.0)
    forall(k: Nat) {
        if q(k) {
            Real.0 <= from_nat[Real](k)
            k.suc = k + Nat.1
            from_nat[Real](k + Nat.1) = from_nat[Real](k) + from_nat[Real](Nat.1)
            from_nat[Real](Nat.1) = Real.1
            from_nat[Real](k.suc) = from_nat[Real](k) + Real.1
            Real.1.is_positive
            Real.0 < Real.1
            Real.0 <= Real.1
            Real.0 <= from_nat[Real](k) + Real.1
            Real.0 <= from_nat[Real](k.suc)
            q(k.suc)
        }
        (q(k) implies q(k.suc))
    }
    q(Nat.0) and forall(k: Nat) {
        q(k) implies q(k.suc)
    }
    Nat.induction(q)
    q(n)
}

/// Embedding the naturals into the reals is order preserving.
///
/// The gap between the two is itself the image of a natural, hence nonnegative.
theorem from_nat_real_lte(m: Nat, n: Nat) {
    m <= n implies from_nat[Real](m) <= from_nat[Real](n)
} by {
    if m <= n {
        let (k: Nat) satisfy {
            m + k = n
        }
        from_nat[Real](m + k) = from_nat[Real](m) + from_nat[Real](k)
        from_nat_real_nonnegative(k)
        Real.0 <= from_nat[Real](k)
        from_nat[Real](m) <= from_nat[Real](m) + from_nat[Real](k)
        from_nat[Real](m) <= from_nat[Real](n)
    }
}

/// Dividing both sides of an inequality by a positive number preserves it.
theorem div_le_div_pos(a: Real, b: Real, c: Real) {
    a <= b and Real.0 < c implies a / c <= b / c
} by {
    if a <= b and Real.0 < c {
        c != Real.0
        mul_div_cancel(a, c)
        c * (a / c) = a
        (a / c) * c = c * (a / c)
        (a / c) * c = a
        (a / c) * c <= b
        div_le_of_mul_le(a / c, c, b)
        a / c <= b / c
    }
}
