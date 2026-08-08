from nat import Nat
from data.basic.functions import function_extensionality
from list import partial, partial_zero, partial_split_last
from real import Real, converges, is_lower_bound, seq_lte, mul_seq, comparison_test,
    geom_converges, converges_mul_seq

/// True if the terms of a sequence are nonnegative.
///
/// The hypothesis every comparison test carries, named so the statements below read as one
/// condition rather than as an unfolded bound.
define is_nonnegative_seq(a: Nat -> Real) -> Bool {
    is_lower_bound(a, Real.0)
}

/// Nonnegativity applies termwise.
theorem is_nonnegative_seq_apply(a: Nat -> Real, n: Nat) {
    is_nonnegative_seq(a) implies Real.0 <= a(n)
} by {
    if is_nonnegative_seq(a) {
        (is_nonnegative_seq(a) = is_lower_bound(a, Real.0))
        is_lower_bound(a, Real.0)
        (is_lower_bound(a, Real.0) = forall(k: Nat) { Real.0 <= a(k) })
        Real.0 <= a(n)
    }
}

/// Termwise nonnegativity gives the predicate.
theorem is_nonnegative_seq_intro(a: Nat -> Real) {
    (forall(n: Nat) { Real.0 <= a(n) }) implies is_nonnegative_seq(a)
} by {
    if forall(n: Nat) { Real.0 <= a(n) } {
        (is_lower_bound(a, Real.0) = forall(k: Nat) { Real.0 <= a(k) })
        is_lower_bound(a, Real.0)
        (is_nonnegative_seq(a) = is_lower_bound(a, Real.0))
        is_nonnegative_seq(a)
    }
}

/// A termwise bound gives the comparison relation.
theorem seq_lte_intro(a: Nat -> Real, b: Nat -> Real) {
    (forall(n: Nat) { a(n) <= b(n) }) implies seq_lte(a, b)
} by {
    if forall(n: Nat) { a(n) <= b(n) } {
        (seq_lte(a, b) = forall(k: Nat) { a(k) <= b(k) })
        seq_lte(a, b)
    }
}

/// A constant multiple of a geometric sequence sums to a convergent series.
///
/// The dominating series of every geometric comparison. `mul_seq` scales termwise and
/// `converges_mul_seq` carries convergence across that scaling, so this needs only the
/// convergence of the geometric series itself.
theorem scaled_geom_converges(c: Real, r: Real) {
    r.abs < Real.1 implies converges(mul_seq(c, partial(r.pow)))
} by {
    if r.abs < Real.1 {
        geom_converges(r)
        converges(partial(r.pow))
        converges_mul_seq(c, partial(r.pow))
        converges(mul_seq(c, partial(r.pow)))
    }
}

/// The partial sums of a scaled sequence are the scaled partial sums.
theorem partial_mul_seq(c: Real, a: Nat -> Real, n: Nat) {
    partial(mul_seq(c, a), n) = mul_seq(c, partial(a), n)
} by {
    define p(x: Nat) -> Bool {
        partial(mul_seq(c, a), x) = mul_seq(c, partial(a), x)
    }
    partial_zero(mul_seq(c, a))
    partial(mul_seq(c, a), Nat.0) = Real.0
    partial_zero(a)
    partial(a, Nat.0) = Real.0
    (mul_seq(c, partial(a), Nat.0) = c * partial(a, Nat.0))
    (c * Real.0 = Real.0)
    p(Nat.0)
    forall(m: Nat) {
        if p(m) {
            partial(mul_seq(c, a), m) = mul_seq(c, partial(a), m)
            partial_split_last(mul_seq(c, a), m)
            (partial(mul_seq(c, a), m.suc) = partial(mul_seq(c, a), m) + mul_seq(c, a, m))
            (mul_seq(c, a, m) = c * a(m))
            (mul_seq(c, partial(a), m) = c * partial(a, m))
            (partial(mul_seq(c, a), m.suc) = c * partial(a, m) + c * a(m))
            (c * partial(a, m) + c * a(m) = c * (partial(a, m) + a(m)))
            partial_split_last(a, m)
            partial(a, m.suc) = partial(a, m) + a(m)
            (partial(mul_seq(c, a), m.suc) = c * partial(a, m.suc))
            (mul_seq(c, partial(a), m.suc) = c * partial(a, m.suc))
            partial(mul_seq(c, a), m.suc) = mul_seq(c, partial(a), m.suc)
            p(m.suc)
        }
        (p(m) implies p(m.suc))
    }
    p(Nat.0) and forall(m: Nat) {
        p(m) implies p(m.suc)
    }
    Nat.induction(p)
    p(n)
}

/// A nonnegative series dominated by a geometric one converges.
///
/// The geometric comparison test. The dominating series is a constant times a geometric
/// sequence, so its partial sums converge, and the comparison test does the rest.
theorem geometric_comparison_test(a: Nat -> Real, c: Real, r: Real) {
    is_nonnegative_seq(a) and r.abs < Real.1
        and (forall(n: Nat) { a(n) <= c * r.pow(n) })
        implies converges(partial(a))
} by {
    if is_nonnegative_seq(a) and r.abs < Real.1
        and forall(n: Nat) { a(n) <= c * r.pow(n) } {
        forall(n: Nat) {
            (a(n) <= c * r.pow(n))
            (mul_seq(c, r.pow, n) = c * r.pow(n))
            a(n) <= mul_seq(c, r.pow, n)
        }
        seq_lte_intro(a, mul_seq(c, r.pow))
        seq_lte(a, mul_seq(c, r.pow))
        forall(n: Nat) {
            partial_mul_seq(c, r.pow, n)
            partial(mul_seq(c, r.pow), n) = mul_seq(c, partial(r.pow), n)
        }
        scaled_geom_converges(c, r)
        converges(mul_seq(c, partial(r.pow)))
        function_extensionality[Nat, Real](partial(mul_seq(c, r.pow)),
            mul_seq(c, partial(r.pow)))
        partial(mul_seq(c, r.pow)) = mul_seq(c, partial(r.pow))
        converges(partial(mul_seq(c, r.pow)))
        (is_nonnegative_seq(a) = is_lower_bound(a, Real.0))
        is_lower_bound(a, Real.0)
        comparison_test(a, mul_seq(c, r.pow))
        converges(partial(a))
    }
}
