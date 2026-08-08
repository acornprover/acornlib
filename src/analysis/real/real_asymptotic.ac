from nat import Nat
from real import Real, add_seq, lte_trans, lte_mul_nonneg_left, mul_abs, mul_pos_pos,
    abs_gte_zero, pos_gt_zero, gt_zero_imp_pos, lt_add_pos, lt_trans, mul_le_mul_nonneg
from algebra.comm_monoid_rearrange import mul_swap_inner
from data.fin.fin_sum_abs import abs_add_le

/// A sum bounded above bounds its left summand.
///
/// Used to combine two thresholds by adding them, which avoids needing a maximum. Stated here
/// because `lte_trans` on the naturals is shadowed by the one on the reals in this module.
theorem lte_of_add_lte(a: Nat, b: Nat, k: Nat) {
    a + b <= k implies a <= k
} by {
    if a + b <= k {
        ((a + b <= k) = exists(c: Nat) { a + b + c = k })
        let (d: Nat) satisfy {
            a + b + d = k
        }
        (a + (b + d) = k)
        exists(e: Nat) { a + e = k }
        ((a <= k) = exists(e: Nat) { a + e = k })
        a <= k
    }
}

/// True if `f` is bounded by `c` times `g` at every index from `n` onward.
///
/// The inner condition of every statement below, named so that the quantifier depth of each
/// definition stays at one. Written with absolute values on both sides, so it says nothing about
/// signs.
define dominated_from(f: Nat -> Real, g: Nat -> Real, c: Real, n: Nat) -> Bool {
    forall(k: Nat) {
        n <= k implies f(k).abs <= c * g(k).abs
    }
}

/// The domination bound holds at each index past the threshold.
theorem dominated_from_apply(
    f: Nat -> Real, g: Nat -> Real, c: Real, n: Nat, k: Nat
) {
    dominated_from(f, g, c, n) and n <= k implies f(k).abs <= c * g(k).abs
} by {
    if dominated_from(f, g, c, n) and n <= k {
        (dominated_from(f, g, c, n) = forall(j: Nat) {
            n <= j implies f(j).abs <= c * g(j).abs
        })
        forall(j: Nat) {
            n <= j implies f(j).abs <= c * g(j).abs
        }
        (n <= k implies f(k).abs <= c * g(k).abs)
        f(k).abs <= c * g(k).abs
    }
}

/// The pointwise bounds give the domination bound.
theorem dominated_from_intro(f: Nat -> Real, g: Nat -> Real, c: Real, n: Nat) {
    (forall(k: Nat) { n <= k implies f(k).abs <= c * g(k).abs })
        implies dominated_from(f, g, c, n)
} by {
    (dominated_from(f, g, c, n) = forall(k: Nat) {
        n <= k implies f(k).abs <= c * g(k).abs
    })
}

/// True if `f` is eventually bounded by a constant multiple of `g`.
///
/// The big-O relation. The constant is required to be positive, which loses nothing and keeps
/// the transitivity argument from needing a sign analysis.
define is_big_o(f: Nat -> Real, g: Nat -> Real) -> Bool {
    exists(c: Real, n: Nat) {
        c.is_positive and dominated_from(f, g, c, n)
    }
}

/// A positive constant and a threshold witness the big-O relation.
theorem is_big_o_intro(f: Nat -> Real, g: Nat -> Real, c: Real, n: Nat) {
    c.is_positive and dominated_from(f, g, c, n) implies is_big_o(f, g)
} by {
    if c.is_positive and dominated_from(f, g, c, n) {
        exists(d: Real, m: Nat) {
            d.is_positive and dominated_from(f, g, d, m)
        }
        (is_big_o(f, g) = exists(d: Real, m: Nat) {
            d.is_positive and dominated_from(f, g, d, m)
        })
        is_big_o(f, g)
    }
}

/// The big-O relation yields a constant and a threshold.
theorem is_big_o_witness(f: Nat -> Real, g: Nat -> Real) {
    is_big_o(f, g) implies exists(c: Real, n: Nat) {
        c.is_positive and dominated_from(f, g, c, n)
    }
} by {
    if is_big_o(f, g) {
        (is_big_o(f, g) = exists(d: Real, m: Nat) {
            d.is_positive and dominated_from(f, g, d, m)
        })
        exists(d: Real, m: Nat) {
            d.is_positive and dominated_from(f, g, d, m)
        }
    }
}

/// Every sequence is big-O of itself.
theorem is_big_o_refl(f: Nat -> Real) {
    is_big_o(f, f)
} by {
    forall(k: Nat) {
        (Real.1 * f(k).abs = f(k).abs)
        f(k).abs <= Real.1 * f(k).abs
        (Nat.0 <= k implies f(k).abs <= Real.1 * f(k).abs)
    }
    dominated_from_intro(f, f, Real.1, Nat.0)
    dominated_from(f, f, Real.1, Nat.0)
    Real.1.is_positive
    is_big_o_intro(f, f, Real.1, Nat.0)
    is_big_o(f, f)
}

/// The big-O relation is transitive.
///
/// The constants multiply and the thresholds are combined by adding them, which dominates both
/// and avoids needing a maximum.
theorem is_big_o_trans(f: Nat -> Real, g: Nat -> Real, h: Nat -> Real) {
    is_big_o(f, g) and is_big_o(g, h) implies is_big_o(f, h)
} by {
    if is_big_o(f, g) and is_big_o(g, h) {
        is_big_o_witness(f, g)
        exists(c: Real, n: Nat) {
            c.is_positive and dominated_from(f, g, c, n)
        }
        let (c1: Real, n1: Nat) satisfy {
            c1.is_positive and dominated_from(f, g, c1, n1)
        }
        is_big_o_witness(g, h)
        exists(c: Real, n: Nat) {
            c.is_positive and dominated_from(g, h, c, n)
        }
        let (c2: Real, n2: Nat) satisfy {
            c2.is_positive and dominated_from(g, h, c2, n2)
        }
        forall(k: Nat) {
            if n1 + n2 <= k {
                lte_of_add_lte(n1, n2, k)
                n1 <= k
                dominated_from_apply(f, g, c1, n1, k)
                f(k).abs <= c1 * g(k).abs
                (n2 + n1 = n1 + n2)
                n2 + n1 <= k
                lte_of_add_lte(n2, n1, k)
                n2 <= k
                dominated_from_apply(g, h, c2, n2, k)
                g(k).abs <= c2 * h(k).abs
                not c1.is_negative
                lte_mul_nonneg_left(g(k).abs, c2 * h(k).abs, c1)
                c1 * g(k).abs <= c1 * (c2 * h(k).abs)
                (c1 * (c2 * h(k).abs) = (c1 * c2) * h(k).abs)
                c1 * g(k).abs <= (c1 * c2) * h(k).abs
                lte_trans(f(k).abs, c1 * g(k).abs, (c1 * c2) * h(k).abs)
                f(k).abs <= (c1 * c2) * h(k).abs
            }
            (n1 + n2 <= k implies f(k).abs <= (c1 * c2) * h(k).abs)
        }
        dominated_from_intro(f, h, c1 * c2, n1 + n2)
        dominated_from(f, h, c1 * c2, n1 + n2)
        mul_pos_pos(c1, c2)
        (c1 * c2).is_positive
        is_big_o_intro(f, h, c1 * c2, n1 + n2)
        is_big_o(f, h)
    }
}

/// A sum of two sequences is big-O of anything both are big-O of.
///
/// The triangle inequality bounds the sum termwise, and the two constants add.
theorem is_big_o_add(f1: Nat -> Real, f2: Nat -> Real, g: Nat -> Real) {
    is_big_o(f1, g) and is_big_o(f2, g) implies is_big_o(add_seq(f1, f2), g)
} by {
    if is_big_o(f1, g) and is_big_o(f2, g) {
        is_big_o_witness(f1, g)
        exists(c: Real, n: Nat) {
            c.is_positive and dominated_from(f1, g, c, n)
        }
        let (c1: Real, n1: Nat) satisfy {
            c1.is_positive and dominated_from(f1, g, c1, n1)
        }
        is_big_o_witness(f2, g)
        exists(c: Real, n: Nat) {
            c.is_positive and dominated_from(f2, g, c, n)
        }
        let (c2: Real, n2: Nat) satisfy {
            c2.is_positive and dominated_from(f2, g, c2, n2)
        }
        forall(k: Nat) {
            if n1 + n2 <= k {
                lte_of_add_lte(n1, n2, k)
                n1 <= k
                dominated_from_apply(f1, g, c1, n1, k)
                f1(k).abs <= c1 * g(k).abs
                (n2 + n1 = n1 + n2)
                n2 + n1 <= k
                lte_of_add_lte(n2, n1, k)
                n2 <= k
                dominated_from_apply(f2, g, c2, n2, k)
                f2(k).abs <= c2 * g(k).abs
                abs_add_le(f1(k), f2(k))
                (f1(k) + f2(k)).abs <= f1(k).abs + f2(k).abs
                (f1(k).abs + f2(k).abs <= c1 * g(k).abs + c2 * g(k).abs)
                (c1 * g(k).abs + c2 * g(k).abs = (c1 + c2) * g(k).abs)
                lte_trans((f1(k) + f2(k)).abs, f1(k).abs + f2(k).abs,
                    (c1 + c2) * g(k).abs)
                (f1(k) + f2(k)).abs <= (c1 + c2) * g(k).abs
                (add_seq(f1, f2)(k) = f1(k) + f2(k))
                add_seq(f1, f2)(k).abs <= (c1 + c2) * g(k).abs
            }
            (n1 + n2 <= k implies add_seq(f1, f2)(k).abs <= (c1 + c2) * g(k).abs)
        }
        dominated_from_intro(add_seq(f1, f2), g, c1 + c2, n1 + n2)
        dominated_from(add_seq(f1, f2), g, c1 + c2, n1 + n2)
        pos_gt_zero(c1)
        c1 > Real.0
        lt_add_pos(c1, c2)
        c1 < c1 + c2
        lt_trans(Real.0, c1, c1 + c2)
        Real.0 < c1 + c2
        c1 + c2 > Real.0
        gt_zero_imp_pos(c1 + c2)
        (c1 + c2).is_positive
        is_big_o_intro(add_seq(f1, f2), g, c1 + c2, n1 + n2)
        is_big_o(add_seq(f1, f2), g)
    }
}

/// A sequence scaled by a constant.
define scale_real_seq(a: Real, f: Nat -> Real) -> (Nat -> Real) {
    function(k: Nat) {
        a * f(k)
    }
}

/// Scaling a sequence does not change what it is big-O of.
theorem is_big_o_scale(a: Real, f: Nat -> Real, g: Nat -> Real) {
    a.is_positive and is_big_o(f, g) implies is_big_o(scale_real_seq(a, f), g)
} by {
    if a.is_positive and is_big_o(f, g) {
        is_big_o_witness(f, g)
        exists(c: Real, n: Nat) {
            c.is_positive and dominated_from(f, g, c, n)
        }
        let (c1: Real, n1: Nat) satisfy {
            c1.is_positive and dominated_from(f, g, c1, n1)
        }
        forall(k: Nat) {
            if n1 <= k {
                dominated_from_apply(f, g, c1, n1, k)
                f(k).abs <= c1 * g(k).abs
                not a.is_negative
                a.abs = a
                mul_abs(a, f(k))
                ((a * f(k)).abs = a.abs * f(k).abs)
                ((a * f(k)).abs = a * f(k).abs)
                lte_mul_nonneg_left(f(k).abs, c1 * g(k).abs, a)
                a * f(k).abs <= a * (c1 * g(k).abs)
                (a * (c1 * g(k).abs) = (a * c1) * g(k).abs)
                (a * f(k)).abs <= (a * c1) * g(k).abs
                (scale_real_seq(a, f)(k) = a * f(k))
                scale_real_seq(a, f)(k).abs <= (a * c1) * g(k).abs
            }
            (n1 <= k implies scale_real_seq(a, f)(k).abs <= (a * c1) * g(k).abs)
        }
        dominated_from_intro(scale_real_seq(a, f), g, a * c1, n1)
        dominated_from(scale_real_seq(a, f), g, a * c1, n1)
        mul_pos_pos(a, c1)
        (a * c1).is_positive
        is_big_o_intro(scale_real_seq(a, f), g, a * c1, n1)
        is_big_o(scale_real_seq(a, f), g)
    }
}

/// True if `f` is eventually bounded by `eps` times `g`.
define little_o_bound(f: Nat -> Real, g: Nat -> Real, eps: Real) -> Bool {
    exists(n: Nat) {
        dominated_from(f, g, eps, n)
    }
}

/// A threshold witnesses the little-o bound at a given size.
theorem little_o_bound_intro(f: Nat -> Real, g: Nat -> Real, eps: Real, n: Nat) {
    dominated_from(f, g, eps, n) implies little_o_bound(f, g, eps)
} by {
    if dominated_from(f, g, eps, n) {
        exists(m: Nat) {
            dominated_from(f, g, eps, m)
        }
        (little_o_bound(f, g, eps) = exists(m: Nat) {
            dominated_from(f, g, eps, m)
        })
        little_o_bound(f, g, eps)
    }
}

/// The little-o bound yields a threshold.
theorem little_o_bound_witness(f: Nat -> Real, g: Nat -> Real, eps: Real) {
    little_o_bound(f, g, eps) implies exists(n: Nat) { dominated_from(f, g, eps, n) }
} by {
    if little_o_bound(f, g, eps) {
        (little_o_bound(f, g, eps) = exists(m: Nat) {
            dominated_from(f, g, eps, m)
        })
        exists(m: Nat) {
            dominated_from(f, g, eps, m)
        }
    }
}

/// True if `f` is eventually bounded by every positive multiple of `g`.
///
/// The little-o relation. Unlike big-O the constant is not fixed but arbitrary, which is what
/// makes it the statement that `f` is negligible against `g`.
define is_little_o(f: Nat -> Real, g: Nat -> Real) -> Bool {
    forall(eps: Real) {
        eps.is_positive implies little_o_bound(f, g, eps)
    }
}

/// The little-o relation gives a bound at each positive size.
theorem is_little_o_apply(f: Nat -> Real, g: Nat -> Real, eps: Real) {
    is_little_o(f, g) and eps.is_positive implies little_o_bound(f, g, eps)
} by {
    if is_little_o(f, g) and eps.is_positive {
        (is_little_o(f, g) = forall(d: Real) {
            d.is_positive implies little_o_bound(f, g, d)
        })
        forall(d: Real) {
            d.is_positive implies little_o_bound(f, g, d)
        }
        (eps.is_positive implies little_o_bound(f, g, eps))
        little_o_bound(f, g, eps)
    }
}

/// Bounds at every positive size give the little-o relation.
theorem is_little_o_intro(f: Nat -> Real, g: Nat -> Real) {
    (forall(eps: Real) { eps.is_positive implies little_o_bound(f, g, eps) })
        implies is_little_o(f, g)
} by {
    (is_little_o(f, g) = forall(eps: Real) {
        eps.is_positive implies little_o_bound(f, g, eps)
    })
}

/// Negligible sequences are in particular boundedly dominated.
///
/// Taking the size to be one turns the little-o bound into a big-O witness.
theorem is_big_o_of_is_little_o(f: Nat -> Real, g: Nat -> Real) {
    is_little_o(f, g) implies is_big_o(f, g)
} by {
    if is_little_o(f, g) {
        Real.1.is_positive
        is_little_o_apply(f, g, Real.1)
        little_o_bound(f, g, Real.1)
        little_o_bound_witness(f, g, Real.1)
        exists(n: Nat) { dominated_from(f, g, Real.1, n) }
        let (n: Nat) satisfy {
            dominated_from(f, g, Real.1, n)
        }
        is_big_o_intro(f, g, Real.1, n)
        is_big_o(f, g)
    }
}

/// The pointwise product of two sequences.
define mul_real_seq(f: Nat -> Real, g: Nat -> Real) -> (Nat -> Real) {
    function(k: Nat) {
        f(k) * g(k)
    }
}

/// Products respect the big-O relation in both factors.
///
/// The termwise bounds multiply, which is where the constants being positive is used a second
/// time: the product bound needs both sides nonnegative.
theorem is_big_o_mul(
    f1: Nat -> Real, g1: Nat -> Real, f2: Nat -> Real, g2: Nat -> Real
) {
    is_big_o(f1, g1) and is_big_o(f2, g2)
        implies is_big_o(mul_real_seq(f1, f2), mul_real_seq(g1, g2))
} by {
    if is_big_o(f1, g1) and is_big_o(f2, g2) {
        is_big_o_witness(f1, g1)
        exists(c: Real, n: Nat) {
            c.is_positive and dominated_from(f1, g1, c, n)
        }
        let (c1: Real, n1: Nat) satisfy {
            c1.is_positive and dominated_from(f1, g1, c1, n1)
        }
        is_big_o_witness(f2, g2)
        exists(c: Real, n: Nat) {
            c.is_positive and dominated_from(f2, g2, c, n)
        }
        let (c2: Real, n2: Nat) satisfy {
            c2.is_positive and dominated_from(f2, g2, c2, n2)
        }
        forall(k: Nat) {
            if n1 + n2 <= k {
                lte_of_add_lte(n1, n2, k)
                n1 <= k
                (n2 + n1 = n1 + n2)
                n2 + n1 <= k
                lte_of_add_lte(n2, n1, k)
                n2 <= k
                dominated_from_apply(f1, g1, c1, n1, k)
                f1(k).abs <= c1 * g1(k).abs
                dominated_from_apply(f2, g2, c2, n2, k)
                f2(k).abs <= c2 * g2(k).abs
                abs_gte_zero(f1(k))
                f1(k).abs >= Real.0
                abs_gte_zero(f2(k))
                f2(k).abs >= Real.0
                pos_gt_zero(c1)
                c1 > Real.0
                abs_gte_zero(g1(k))
                g1(k).abs >= Real.0
                c1 * g1(k).abs >= Real.0
                pos_gt_zero(c2)
                c2 > Real.0
                abs_gte_zero(g2(k))
                g2(k).abs >= Real.0
                c2 * g2(k).abs >= Real.0
                mul_le_mul_nonneg(f1(k).abs, f2(k).abs, c1 * g1(k).abs, c2 * g2(k).abs)
                (f1(k).abs * f2(k).abs <= (c1 * g1(k).abs) * (c2 * g2(k).abs))
                mul_swap_inner[Real](c1, g1(k).abs, c2, g2(k).abs)
                ((c1 * g1(k).abs) * (c2 * g2(k).abs)
                    = (c1 * c2) * (g1(k).abs * g2(k).abs))
                mul_abs(f1(k), f2(k))
                ((f1(k) * f2(k)).abs = f1(k).abs * f2(k).abs)
                mul_abs(g1(k), g2(k))
                ((g1(k) * g2(k)).abs = g1(k).abs * g2(k).abs)
                ((f1(k) * f2(k)).abs <= (c1 * c2) * (g1(k) * g2(k)).abs)
                (mul_real_seq(f1, f2)(k) = f1(k) * f2(k))
                (mul_real_seq(g1, g2)(k) = g1(k) * g2(k))
                (mul_real_seq(f1, f2)(k).abs
                    <= (c1 * c2) * mul_real_seq(g1, g2)(k).abs)
            }
            (n1 + n2 <= k implies mul_real_seq(f1, f2)(k).abs
                <= (c1 * c2) * mul_real_seq(g1, g2)(k).abs)
        }
        dominated_from_intro(mul_real_seq(f1, f2), mul_real_seq(g1, g2), c1 * c2, n1 + n2)
        dominated_from(mul_real_seq(f1, f2), mul_real_seq(g1, g2), c1 * c2, n1 + n2)
        mul_pos_pos(c1, c2)
        (c1 * c2).is_positive
        is_big_o_intro(mul_real_seq(f1, f2), mul_real_seq(g1, g2), c1 * c2, n1 + n2)
        is_big_o(mul_real_seq(f1, f2), mul_real_seq(g1, g2))
    }
}

/// True if each of two sequences is big-O of the other.
///
/// The big-Theta relation: the two grow at the same rate up to constants.
define is_big_theta(f: Nat -> Real, g: Nat -> Real) -> Bool {
    is_big_o(f, g) and is_big_o(g, f)
}

/// Big-Theta gives both big-O relations.
theorem is_big_theta_apply(f: Nat -> Real, g: Nat -> Real) {
    is_big_theta(f, g) implies is_big_o(f, g) and is_big_o(g, f)
} by {
    if is_big_theta(f, g) {
        (is_big_theta(f, g) = (is_big_o(f, g) and is_big_o(g, f)))
        (is_big_o(f, g) and is_big_o(g, f))
    }
}

/// Both big-O relations give big-Theta.
theorem is_big_theta_intro(f: Nat -> Real, g: Nat -> Real) {
    is_big_o(f, g) and is_big_o(g, f) implies is_big_theta(f, g)
} by {
    if is_big_o(f, g) and is_big_o(g, f) {
        (is_big_theta(f, g) = (is_big_o(f, g) and is_big_o(g, f)))
        is_big_theta(f, g)
    }
}

/// Every sequence grows at its own rate.
theorem is_big_theta_refl(f: Nat -> Real) {
    is_big_theta(f, f)
} by {
    is_big_o_refl(f)
    is_big_o(f, f)
    is_big_theta_intro(f, f)
    is_big_theta(f, f)
}

/// Growing at the same rate is symmetric.
theorem is_big_theta_symm(f: Nat -> Real, g: Nat -> Real) {
    is_big_theta(f, g) implies is_big_theta(g, f)
} by {
    if is_big_theta(f, g) {
        is_big_theta_apply(f, g)
        (is_big_o(f, g) and is_big_o(g, f))
        is_big_theta_intro(g, f)
        is_big_theta(g, f)
    }
}

/// Growing at the same rate is transitive.
///
/// With reflexivity and symmetry this makes big-Theta an equivalence relation on sequences.
theorem is_big_theta_trans(f: Nat -> Real, g: Nat -> Real, h: Nat -> Real) {
    is_big_theta(f, g) and is_big_theta(g, h) implies is_big_theta(f, h)
} by {
    if is_big_theta(f, g) and is_big_theta(g, h) {
        is_big_theta_apply(f, g)
        (is_big_o(f, g) and is_big_o(g, f))
        is_big_theta_apply(g, h)
        (is_big_o(g, h) and is_big_o(h, g))
        is_big_o_trans(f, g, h)
        is_big_o(f, h)
        is_big_o_trans(h, g, f)
        is_big_o(h, f)
        is_big_theta_intro(f, h)
        is_big_theta(f, h)
    }
}
