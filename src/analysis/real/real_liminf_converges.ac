from nat import Nat
from real import Real, converges_to, tail_bound, tail_bound_implies_is_close, close_imp_bounds,
    lte_antisymm, lte_trans, lte_lt_trans, lte_add_right, eventual_ub, ub_imp_limit_lte, limit,
    converges
from analysis.real.real_tail_supremum import is_bounded_above
from analysis.real.real_limsup import is_bounded_below, neg_tail_sup, neg_tail_sup_converges
from analysis.real.real_liminf import liminf, tail_inf, tail_inf_bounds_terms, tail_inf_is_greatest,
    liminf_ge_tail_inf, liminf_eq_limit_tail_inf, neg_tail_sup_neg_seq_eq_tail_inf, neg_seq,
    neg_seq_bounded_above, neg_seq_bounded_below
from analysis.real.real_limsup_converges import lte_of_lte_add_eps

/// The limit of a convergent sequence is at most its limit inferior.
///
/// Past the threshold every term is above the limit minus a size, so the tail infimum there is
/// too, and the limit inferior is above that tail infimum.
theorem limit_le_liminf(a: Nat -> Real, l: Real) {
    is_bounded_above(a) and is_bounded_below(a) and converges_to(a, l)
        implies l <= liminf(a)
} by {
    if is_bounded_above(a) and is_bounded_below(a) and converges_to(a, l) {
        forall(eps: Real) {
            if eps.is_positive {
                (converges_to(a, l) = forall(d: Real) {
                    d.is_positive implies exists(m: Nat) { tail_bound(a, l, m, d) }
                })
                exists(m: Nat) { tail_bound(a, l, m, eps) }
                let (n: Nat) satisfy {
                    tail_bound(a, l, n, eps)
                }
                forall(k: Nat) {
                    if n <= k {
                        tail_bound_implies_is_close(a, l, n, eps, k)
                        a(k).is_close(l, eps)
                        close_imp_bounds(a(k), l, eps)
                        l - eps < a(k)
                        l - eps <= a(k)
                    }
                    (n <= k implies l - eps <= a(k))
                }
                tail_inf_is_greatest(a, n, l - eps)
                l - eps <= tail_inf(a, n)
                liminf_ge_tail_inf(a, n)
                tail_inf(a, n) <= liminf(a)
                lte_trans(l - eps, tail_inf(a, n), liminf(a))
                l - eps <= liminf(a)
                lte_add_right(l - eps, liminf(a), eps)
                (l - eps) + eps <= liminf(a) + eps
                ((l - eps) + eps = l)
                (l <= liminf(a) + eps)
            }
            (eps.is_positive implies l <= liminf(a) + eps)
        }
        lte_of_lte_add_eps(l, liminf(a))
        l <= liminf(a)
    }
}

/// The limit inferior of a convergent sequence is at most its limit.
///
/// Every tail infimum lies below a term past the threshold, hence below the limit plus a size,
/// so their limit does too.
theorem liminf_le_limit(a: Nat -> Real, l: Real) {
    is_bounded_above(a) and is_bounded_below(a) and converges_to(a, l)
        implies liminf(a) <= l
} by {
    if is_bounded_above(a) and is_bounded_below(a) and converges_to(a, l) {
        neg_seq_bounded_above(a)
        is_bounded_above(neg_seq(a))
        neg_seq_bounded_below(a)
        is_bounded_below(neg_seq(a))
        neg_tail_sup_converges(neg_seq(a))
        converges(neg_tail_sup(neg_seq(a)))
        forall(eps: Real) {
            if eps.is_positive {
                (converges_to(a, l) = forall(d: Real) {
                    d.is_positive implies exists(m: Nat) { tail_bound(a, l, m, d) }
                })
                exists(m: Nat) { tail_bound(a, l, m, eps) }
                let (n: Nat) satisfy {
                    tail_bound(a, l, n, eps)
                }
                forall(j: Nat) {
                    if Nat.0 <= j {
                        n <= j + n
                        tail_bound_implies_is_close(a, l, n, eps, j + n)
                        a(j + n).is_close(l, eps)
                        close_imp_bounds(a(j + n), l, eps)
                        a(j + n) < l + eps
                        j <= j + n
                        tail_inf_bounds_terms(a, j, j + n)
                        tail_inf(a, j) <= a(j + n)
                        lte_lt_trans(tail_inf(a, j), a(j + n), l + eps)
                        tail_inf(a, j) < l + eps
                        tail_inf(a, j) <= l + eps
                        neg_tail_sup_neg_seq_eq_tail_inf(a, j)
                        neg_tail_sup(neg_seq(a))(j) = tail_inf(a, j)
                        neg_tail_sup(neg_seq(a))(j) <= l + eps
                    }
                    (Nat.0 <= j implies neg_tail_sup(neg_seq(a))(j) <= l + eps)
                }
                (eventual_ub(neg_tail_sup(neg_seq(a)), l + eps) = exists(m: Nat) {
                    forall(i: Nat) {
                        m <= i implies neg_tail_sup(neg_seq(a))(i) <= l + eps
                    }
                })
                exists(m: Nat) {
                    forall(i: Nat) {
                        m <= i implies neg_tail_sup(neg_seq(a))(i) <= l + eps
                    }
                }
                eventual_ub(neg_tail_sup(neg_seq(a)), l + eps)
                ub_imp_limit_lte(neg_tail_sup(neg_seq(a)), l + eps)
                limit(neg_tail_sup(neg_seq(a))) <= l + eps
                liminf_eq_limit_tail_inf(a)
                liminf(a) = limit(neg_tail_sup(neg_seq(a)))
                liminf(a) <= l + eps
            }
            (eps.is_positive implies liminf(a) <= l + eps)
        }
        lte_of_lte_add_eps(liminf(a), l)
        liminf(a) <= l
    }
}

/// The limit inferior of a convergent sequence is its limit.
theorem liminf_eq_limit(a: Nat -> Real, l: Real) {
    is_bounded_above(a) and is_bounded_below(a) and converges_to(a, l)
        implies liminf(a) = l
} by {
    if is_bounded_above(a) and is_bounded_below(a) and converges_to(a, l) {
        liminf_le_limit(a, l)
        liminf(a) <= l
        limit_le_liminf(a, l)
        l <= liminf(a)
        lte_antisymm(liminf(a), l)
        liminf(a) = l
    }
}
