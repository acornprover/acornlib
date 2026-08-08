from nat import Nat
from real import Real, is_upper_bound, is_lower_bound, converges, limit, eventual_ub,
    ub_imp_limit_lte, lte_trans
from analysis.real.real_tail_supremum import tail_sup, tail_sup_bounds_terms, tail_sup_is_least,
    is_bounded_above, is_bounded_above_intro, is_bounded_above_witness
from analysis.real.real_limsup import limsup, limsup_le_tail_sup, neg_reverses_lte, is_bounded_below,
    is_bounded_below_intro, is_bounded_below_witness, neg_tail_sup, neg_tail_sup_converges

/// The termwise negation of a sequence.
define neg_seq(a: Nat -> Real) -> (Nat -> Real) {
    function(n: Nat) {
        -a(n)
    }
}

/// A sequence bounded below has a negation bounded above.
theorem neg_seq_bounded_above(a: Nat -> Real) {
    is_bounded_below(a) implies is_bounded_above(neg_seq(a))
} by {
    if is_bounded_below(a) {
        is_bounded_below_witness(a)
        exists(b: Real) { is_lower_bound(a, b) }
        let (b: Real) satisfy {
            is_lower_bound(a, b)
        }
        (is_lower_bound(a, b) = forall(k: Nat) { b <= a(k) })
        forall(n: Nat) {
            b <= a(n)
            neg_reverses_lte(b, a(n))
            -a(n) <= -b
            (neg_seq(a)(n) = -a(n))
            neg_seq(a)(n) <= -b
        }
        (is_upper_bound(neg_seq(a), -b) = forall(n: Nat) { neg_seq(a)(n) <= -b })
        is_upper_bound(neg_seq(a), -b)
        is_bounded_above_intro(neg_seq(a), -b)
        is_bounded_above(neg_seq(a))
    }
}

/// A sequence bounded above has a negation bounded below.
theorem neg_seq_bounded_below(a: Nat -> Real) {
    is_bounded_above(a) implies is_bounded_below(neg_seq(a))
} by {
    if is_bounded_above(a) {
        is_bounded_above_witness(a)
        exists(b: Real) { is_upper_bound(a, b) }
        let (b: Real) satisfy {
            is_upper_bound(a, b)
        }
        (is_upper_bound(a, b) = forall(k: Nat) { a(k) <= b })
        forall(n: Nat) {
            a(n) <= b
            neg_reverses_lte(a(n), b)
            -b <= -a(n)
            (neg_seq(a)(n) = -a(n))
            -b <= neg_seq(a)(n)
        }
        (is_lower_bound(neg_seq(a), -b) = forall(n: Nat) { -b <= neg_seq(a)(n) })
        is_lower_bound(neg_seq(a), -b)
        is_bounded_below_intro(neg_seq(a), -b)
        is_bounded_below(neg_seq(a))
    }
}

/// The greatest bound below the values a sequence takes from index `n` onward.
///
/// The mirror of `tail_sup`, obtained by negating rather than by repeating the construction with
/// infima, which `src/real/` does not supply an existence theorem for.
define tail_inf(a: Nat -> Real, n: Nat) -> Real {
    -tail_sup(neg_seq(a), n)
}

/// No term from the index onward falls below the tail infimum.
theorem tail_inf_bounds_terms(a: Nat -> Real, n: Nat, k: Nat) {
    is_bounded_below(a) and n <= k implies tail_inf(a, n) <= a(k)
} by {
    if is_bounded_below(a) and n <= k {
        neg_seq_bounded_above(a)
        is_bounded_above(neg_seq(a))
        tail_sup_bounds_terms(neg_seq(a), n, k)
        neg_seq(a)(k) <= tail_sup(neg_seq(a), n)
        (neg_seq(a)(k) = -a(k))
        -a(k) <= tail_sup(neg_seq(a), n)
        neg_reverses_lte(-a(k), tail_sup(neg_seq(a), n))
        -tail_sup(neg_seq(a), n) <= -(-a(k))
        (-(-a(k)) = a(k))
        (tail_inf(a, n) = -tail_sup(neg_seq(a), n))
        tail_inf(a, n) <= a(k)
    }
}

/// Nothing above the tail infimum is below the whole tail.
theorem tail_inf_is_greatest(a: Nat -> Real, n: Nat, b: Real) {
    is_bounded_below(a) and (forall(k: Nat) { n <= k implies b <= a(k) })
        implies b <= tail_inf(a, n)
} by {
    if is_bounded_below(a) and forall(k: Nat) { n <= k implies b <= a(k) } {
        neg_seq_bounded_above(a)
        is_bounded_above(neg_seq(a))
        forall(k: Nat) {
            if n <= k {
                (n <= k implies b <= a(k))
                b <= a(k)
                neg_reverses_lte(b, a(k))
                -a(k) <= -b
                (neg_seq(a)(k) = -a(k))
                neg_seq(a)(k) <= -b
            }
            (n <= k implies neg_seq(a)(k) <= -b)
        }
        tail_sup_is_least(neg_seq(a), n, -b)
        tail_sup(neg_seq(a), n) <= -b
        neg_reverses_lte(tail_sup(neg_seq(a), n), -b)
        -(-b) <= -tail_sup(neg_seq(a), n)
        (-(-b) = b)
        (tail_inf(a, n) = -tail_sup(neg_seq(a), n))
        b <= tail_inf(a, n)
    }
}

/// The limit inferior of a sequence.
///
/// Where the tail infima settle, obtained as the reflection of the limit superior of the negated
/// sequence. Every property below transfers along that reflection rather than being reproved.
define liminf(a: Nat -> Real) -> Real {
    -limsup(neg_seq(a))
}

/// The limit inferior is at least every tail infimum.
theorem liminf_ge_tail_inf(a: Nat -> Real, n: Nat) {
    is_bounded_above(a) and is_bounded_below(a) implies tail_inf(a, n) <= liminf(a)
} by {
    if is_bounded_above(a) and is_bounded_below(a) {
        neg_seq_bounded_above(a)
        is_bounded_above(neg_seq(a))
        neg_seq_bounded_below(a)
        is_bounded_below(neg_seq(a))
        limsup_le_tail_sup(neg_seq(a), n)
        limsup(neg_seq(a)) <= tail_sup(neg_seq(a), n)
        neg_reverses_lte(limsup(neg_seq(a)), tail_sup(neg_seq(a), n))
        -tail_sup(neg_seq(a), n) <= -limsup(neg_seq(a))
        (tail_inf(a, n) = -tail_sup(neg_seq(a), n))
        (liminf(a) = -limsup(neg_seq(a)))
        tail_inf(a, n) <= liminf(a)
    }
}

/// Every tail infimum is below every tail supremum.
///
/// The two tails overlap from the later of the two indices onward, and a term there is above the
/// one and below the other. The index `m + n` serves as that later index.
theorem tail_inf_le_tail_sup(a: Nat -> Real, m: Nat, n: Nat) {
    is_bounded_above(a) and is_bounded_below(a) implies tail_inf(a, m) <= tail_sup(a, n)
} by {
    if is_bounded_above(a) and is_bounded_below(a) {
        m <= m + n
        tail_inf_bounds_terms(a, m, m + n)
        tail_inf(a, m) <= a(m + n)
        n <= m + n
        tail_sup_bounds_terms(a, n, m + n)
        a(m + n) <= tail_sup(a, n)
        lte_trans(tail_inf(a, m), a(m + n), tail_sup(a, n))
        tail_inf(a, m) <= tail_sup(a, n)
    }
}

/// The limit inferior is the limit of the tail infima.
///
/// Unfolding the two reflections: the negated tail suprema of the negated sequence are the tail
/// infima of the sequence itself.
theorem liminf_eq_limit_tail_inf(a: Nat -> Real) {
    liminf(a) = limit(neg_tail_sup(neg_seq(a)))
} by {
    (limsup(neg_seq(a)) = -limit(neg_tail_sup(neg_seq(a))))
    (liminf(a) = -limsup(neg_seq(a)))
    (liminf(a) = -(-limit(neg_tail_sup(neg_seq(a)))))
    ((-(-limit(neg_tail_sup(neg_seq(a))))) = limit(neg_tail_sup(neg_seq(a))))
    liminf(a) = limit(neg_tail_sup(neg_seq(a)))
}

/// The tail infima are the terms whose limit is the limit inferior.
theorem neg_tail_sup_neg_seq_eq_tail_inf(a: Nat -> Real, k: Nat) {
    neg_tail_sup(neg_seq(a))(k) = tail_inf(a, k)
} by {
    (neg_tail_sup(neg_seq(a))(k) = -tail_sup(neg_seq(a), k))
    (tail_inf(a, k) = -tail_sup(neg_seq(a), k))
    neg_tail_sup(neg_seq(a))(k) = tail_inf(a, k)
}

/// The limit inferior is below every tail supremum.
///
/// Every tail infimum is, so the tail supremum bounds the whole sequence of them, and hence
/// their limit.
theorem liminf_le_tail_sup(a: Nat -> Real, n: Nat) {
    is_bounded_above(a) and is_bounded_below(a) implies liminf(a) <= tail_sup(a, n)
} by {
    if is_bounded_above(a) and is_bounded_below(a) {
        neg_seq_bounded_above(a)
        is_bounded_above(neg_seq(a))
        neg_seq_bounded_below(a)
        is_bounded_below(neg_seq(a))
        neg_tail_sup_converges(neg_seq(a))
        converges(neg_tail_sup(neg_seq(a)))
        forall(k: Nat) {
            if Nat.0 <= k {
                tail_inf_le_tail_sup(a, k, n)
                tail_inf(a, k) <= tail_sup(a, n)
                neg_tail_sup_neg_seq_eq_tail_inf(a, k)
                neg_tail_sup(neg_seq(a))(k) = tail_inf(a, k)
                neg_tail_sup(neg_seq(a))(k) <= tail_sup(a, n)
            }
            (Nat.0 <= k implies neg_tail_sup(neg_seq(a))(k) <= tail_sup(a, n))
        }
        (eventual_ub(neg_tail_sup(neg_seq(a)), tail_sup(a, n)) = exists(m: Nat) {
            forall(i: Nat) {
                m <= i implies neg_tail_sup(neg_seq(a))(i) <= tail_sup(a, n)
            }
        })
        exists(m: Nat) {
            forall(i: Nat) {
                m <= i implies neg_tail_sup(neg_seq(a))(i) <= tail_sup(a, n)
            }
        }
        eventual_ub(neg_tail_sup(neg_seq(a)), tail_sup(a, n))
        ub_imp_limit_lte(neg_tail_sup(neg_seq(a)), tail_sup(a, n))
        limit(neg_tail_sup(neg_seq(a))) <= tail_sup(a, n)
        liminf_eq_limit_tail_inf(a)
        liminf(a) = limit(neg_tail_sup(neg_seq(a)))
        liminf(a) <= tail_sup(a, n)
    }
}

/// The limit inferior never exceeds the limit superior.
///
/// The limit inferior is below every tail supremum, so its negation bounds the negated tail
/// suprema and hence their limit, which is the negation of the limit superior.
theorem liminf_le_limsup(a: Nat -> Real) {
    is_bounded_above(a) and is_bounded_below(a) implies liminf(a) <= limsup(a)
} by {
    if is_bounded_above(a) and is_bounded_below(a) {
        neg_tail_sup_converges(a)
        converges(neg_tail_sup(a))
        forall(k: Nat) {
            if Nat.0 <= k {
                liminf_le_tail_sup(a, k)
                liminf(a) <= tail_sup(a, k)
                neg_reverses_lte(liminf(a), tail_sup(a, k))
                -tail_sup(a, k) <= -liminf(a)
                (neg_tail_sup(a)(k) = -tail_sup(a, k))
                neg_tail_sup(a)(k) <= -liminf(a)
            }
            (Nat.0 <= k implies neg_tail_sup(a)(k) <= -liminf(a))
        }
        (eventual_ub(neg_tail_sup(a), -liminf(a)) = exists(m: Nat) {
            forall(i: Nat) {
                m <= i implies neg_tail_sup(a)(i) <= -liminf(a)
            }
        })
        exists(m: Nat) {
            forall(i: Nat) {
                m <= i implies neg_tail_sup(a)(i) <= -liminf(a)
            }
        }
        eventual_ub(neg_tail_sup(a), -liminf(a))
        ub_imp_limit_lte(neg_tail_sup(a), -liminf(a))
        limit(neg_tail_sup(a)) <= -liminf(a)
        neg_reverses_lte(limit(neg_tail_sup(a)), -liminf(a))
        (-(-liminf(a)) <= -limit(neg_tail_sup(a)))
        ((-(-liminf(a))) = liminf(a))
        (limsup(a) = -limit(neg_tail_sup(a)))
        liminf(a) <= limsup(a)
    }
}
