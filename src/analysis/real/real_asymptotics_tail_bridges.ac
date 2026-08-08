/// Explicit-tail bridge lemmas for published real-sequence asymptotics.
///
/// This module is a deliberately small consumer of the accepted `nat_eventually`
/// and `real_asymptotics` APIs. It turns concrete tail witnesses of the form
/// `forall n >= n0, ...` into the public sequence-asymptotic predicates without
/// defining a new generic eventual predicate or editing the real interface.

from nat import Nat
from data.nat.nat_eventually import eventually_after_nat, eventually_predicate_nat,
    eventually_predicate_nat_intro
from real import Real, seq_lte
from analysis.real.real_asymptotics import eventually_seq_lte, eventually_seq_eq,
    is_big_o_seq, big_o_seq_intro

/// An explicit tail pointwise order witness gives eventual sequence order.
theorem eventually_seq_lte_of_eventual_index_lte(a: Nat -> Real, b: Nat -> Real, n0: Nat) {
    (forall(n: Nat) { n0 <= n implies a(n) <= b(n) }) implies eventually_seq_lte(a, b)
} by {
    if forall(n: Nat) { n0 <= n implies a(n) <= b(n) } {
        forall(n: Nat) {
            if n0 <= n {
                a(n) <= b(n)
                function(k: Nat) { a(k) <= b(k) }(n)
            }
        }
        eventually_after_nat(function(k: Nat) { a(k) <= b(k) }, n0)
        eventually_predicate_nat_intro(function(k: Nat) { a(k) <= b(k) }, n0)
        eventually_predicate_nat(function(k: Nat) { a(k) <= b(k) })
        eventually_seq_lte(a, b) = eventually_predicate_nat(function(k: Nat) { a(k) <= b(k) })
        eventually_seq_lte(a, b)
    }
}

/// Pointwise sequence order gives eventual sequence order.
theorem eventually_seq_lte_of_seq_lte(a: Nat -> Real, b: Nat -> Real) {
    seq_lte(a, b) implies eventually_seq_lte(a, b)
} by {
    if seq_lte(a, b) {
        seq_lte(a, b) = forall(n: Nat) {
            a(n) <= b(n)
        }
        forall(n: Nat) {
            if Nat.0 <= n {
                a(n) <= b(n)
            }
        }
        eventually_seq_lte_of_eventual_index_lte(a, b, Nat.0)
        eventually_seq_lte(a, b)
    }
}

/// An explicit tail equality witness gives eventual sequence equality.
theorem eventually_seq_eq_of_eventual_index_eq(a: Nat -> Real, b: Nat -> Real, n0: Nat) {
    (forall(n: Nat) { n0 <= n implies a(n) = b(n) }) implies eventually_seq_eq(a, b)
} by {
    if forall(n: Nat) { n0 <= n implies a(n) = b(n) } {
        forall(n: Nat) {
            if n0 <= n {
                a(n) = b(n)
                function(k: Nat) { a(k) = b(k) }(n)
            }
        }
        eventually_after_nat(function(k: Nat) { a(k) = b(k) }, n0)
        eventually_predicate_nat_intro(function(k: Nat) { a(k) = b(k) }, n0)
        eventually_predicate_nat(function(k: Nat) { a(k) = b(k) })
        eventually_seq_eq(a, b) = eventually_predicate_nat(function(k: Nat) { a(k) = b(k) })
        eventually_seq_eq(a, b)
    }
}

/// An explicit tail absolute-value bound gives big-O with the same positive constant.
theorem big_o_seq_of_eventual_index_abs_lte(a: Nat -> Real, b: Nat -> Real, c: Real, n0: Nat) {
    c.is_positive and (forall(n: Nat) { n0 <= n implies a(n).abs <= c * b(n).abs })
    implies is_big_o_seq(a, b)
} by {
    if c.is_positive and (forall(n: Nat) { n0 <= n implies a(n).abs <= c * b(n).abs }) {
        forall(n: Nat) {
            if n0 <= n {
                a(n).abs <= c * b(n).abs
                function(k: Nat) { a(k).abs <= c * b(k).abs }(n)
            }
        }
        eventually_after_nat(function(k: Nat) { a(k).abs <= c * b(k).abs }, n0)
        eventually_predicate_nat_intro(function(k: Nat) { a(k).abs <= c * b(k).abs }, n0)
        eventually_predicate_nat(function(k: Nat) { a(k).abs <= c * b(k).abs })
        big_o_seq_intro(a, b, c)
        is_big_o_seq(a, b)
    }
}
