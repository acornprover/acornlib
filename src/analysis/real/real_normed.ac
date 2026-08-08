from real import Real
from real import mul_abs
from real import triangle_ineq
from algebra.module.normed_add_comm_group import NormedAddCommGroup
from algebra.module.normed_field import NormedField
numerals Real

/// The norm on `Real` used for the `NormedAddCommGroup` instance.
define real_norm(x: Real) -> Real {
    x.abs
}

/// The norm of a real number agrees with its absolute value.
theorem real_norm_eq_abs(x: Real) {
    real_norm(x) = x.abs
}

/// The norm satisfies the triangle inequality on `Real`.
theorem real_norm_triangle(x: Real, y: Real) {
    real_norm(x + y) <= real_norm(x) + real_norm(y)
} by {
    triangle_ineq(x, y)
    (x + y).abs <= x.abs + y.abs
    real_norm(x + y) = (x + y).abs
    real_norm(x) = x.abs
    real_norm(y) = y.abs
}

/// The absolute value gives `Real` the structure of a normed additive commutative group.
instance Real: NormedAddCommGroup {
    let norm: Real -> Real = real_norm
}

/// The norm on `Real` is multiplicative, making `Real` a normed field.
theorem real_norm_mul(a: Real, b: Real) {
    (a * b).norm = a.norm * b.norm
} by {
    (a * b).norm = (a * b).abs
    mul_abs(a, b)
    a.abs * b.abs = (a * b).abs
    a.norm = a.abs
    b.norm = b.abs
}

/// The absolute value gives `Real` the structure of a normed field.
instance Real: NormedField
