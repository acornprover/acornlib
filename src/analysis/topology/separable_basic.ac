from data.basic.set import Set, subset_contains, union_contains_cases, union_contains_left, union_contains_right
from analysis.topology.topological_space import TopologicalSpace, closure, closure_union
from data.cardinal.countable import is_countable, union_of_countable_is_countable
from analysis.topology.separable import is_subspace_separable

/// Binary union is monotone in both arguments.
lemma union_subset_union_of_subset[T](a: Set[T], b: Set[T], c: Set[T], d: Set[T]) {
    a.subset(c) and b.subset(d) implies a.union(b).subset(c.union(d))
} by {
    if a.subset(c) and b.subset(d) {
        forall(x: T) {
            if a.union(b).contains(x) {
                union_contains_cases(a, b, x)
                if a.contains(x) {
                    subset_contains(a, c, x)
                    c.contains(x)
                    union_contains_left(c, d, x)
                    c.union(d).contains(x)
                } else {
                    b.contains(x)
                    subset_contains(b, d, x)
                    d.contains(x)
                    union_contains_right(c, d, x)
                    c.union(d).contains(x)
                }
            }
        }
    }
}

/// Closure of a union contains any union of sets contained in the corresponding closures.
lemma union_subset_closure_union_of_closure_subset[T: TopologicalSpace](
    s: Set[T],
    t: Set[T],
    ds: Set[T],
    dt: Set[T]
) {
    s.subset(closure(ds)) and t.subset(closure(dt)) implies
        s.union(t).subset(closure(ds.union(dt)))
} by {
    if s.subset(closure(ds)) and t.subset(closure(dt)) {
        closure_union[T](ds, dt)
        closure(ds.union(dt)) = closure(ds).union(closure(dt))
        union_subset_union_of_subset[T](s, t, closure(ds), closure(dt))
        s.union(t).subset(closure(ds).union(closure(dt)))
        s.union(t).subset(closure(ds.union(dt)))
    }
}

/// A binary union of separable subspaces is separable.
theorem separable_union_of_separable[T: TopologicalSpace](s: Set[T], t: Set[T]) {
    is_subspace_separable[T](s) and is_subspace_separable[T](t) implies
        is_subspace_separable[T](s.union(t))
} by {
    if is_subspace_separable[T](s) and is_subspace_separable[T](t) {
        is_subspace_separable[T](s) = exists(ds: Set[T]) {
            ds.subset(s) and is_countable[T](ds) and s.subset(closure(ds))
        }
        is_subspace_separable[T](t) = exists(dt: Set[T]) {
            dt.subset(t) and is_countable[T](dt) and t.subset(closure(dt))
        }
        let ds: Set[T] satisfy {
            ds.subset(s) and is_countable[T](ds) and s.subset(closure(ds))
        }
        let dt: Set[T] satisfy {
            dt.subset(t) and is_countable[T](dt) and t.subset(closure(dt))
        }
        ds.subset(s)
        is_countable[T](ds)
        s.subset(closure(ds))
        dt.subset(t)
        is_countable[T](dt)
        t.subset(closure(dt))
        union_subset_union_of_subset[T](ds, dt, s, t)
        ds.union(dt).subset(s.union(t))
        union_of_countable_is_countable(ds, dt)
        is_countable[T](ds.union(dt))
        union_subset_closure_union_of_closure_subset[T](s, t, ds, dt)
        s.union(t).subset(closure(ds.union(dt)))
        let d = ds.union(dt)
        d.subset(s.union(t))
        is_countable[T](d)
        s.union(t).subset(closure(d))
        d.subset(s.union(t)) and is_countable[T](d) and s.union(t).subset(closure(d))
        exists(e: Set[T]) {
            e.subset(s.union(t)) and is_countable[T](e) and s.union(t).subset(closure(e))
        }
        is_subspace_separable[T](s.union(t))
    }
}
