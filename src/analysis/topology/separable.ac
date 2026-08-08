from data.basic.set import Set
from analysis.topology.topological_space import TopologicalSpace, closure
from data.cardinal.countable import is_countable

/// True if the subspace on `s` has a countable subset whose closure contains `s`.
define is_subspace_separable[T: TopologicalSpace](s: Set[T]) -> Bool {
    exists(d: Set[T]) {
        d.subset(s) and is_countable[T](d) and s.subset(closure(d))
    }
}
