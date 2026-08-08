from data.basic.set import Set, set_is_disjoint_compl, union_compl_is_universal,
    universal_set_compl_is_empty
from analysis.topology.topological_space import is_connected, is_connected_iff,
    is_connected_via_clopen, is_connected_via_clopen_iff,
    is_clopen, is_clopen_imp_open, is_clopen_intro,
    is_disconnection, is_disconnection_intro, is_disconnection_iff
from logic.compl_universal import compl_empty_imp_universal
from logic.compl_partition import compl_eq_of_disjoint_cover

/// A connected space (no disconnection) has only the trivial clopen sets.
theorem connected_imp_via_clopen[T](open: Set[T] -> Bool) {
    is_connected[T](open) implies is_connected_via_clopen[T](open)
} by {
    if is_connected[T](open) {
        is_connected_iff[T](open)
        is_connected_via_clopen_iff[T](open)
        forall(s: Set[T]) {
            if is_clopen[T](open, s) {
                is_clopen_imp_open[T](open, s)
                if s != Set[T].empty_set and s != Set[T].universal_set {
                    if s.c = Set[T].empty_set {
                        compl_empty_imp_universal[T](s)
                        false
                    }
                    set_is_disjoint_compl[T](s)
                    union_compl_is_universal[T](s)
                    is_disconnection_intro[T](open, s, s.c)
                    false
                }
                s = Set[T].empty_set or s = Set[T].universal_set
            }
        }
    }
}

/// From a disconnection, the left piece is open.
theorem disconnection_imp_open_left[T](open: Set[T] -> Bool, u: Set[T], v: Set[T]) {
    is_disconnection[T](open, u, v) implies open(u)
} by {
    if is_disconnection[T](open, u, v) {
        is_disconnection_iff[T](open, u, v)
        if not open(u) {
            false
        }
    }
}

/// From a disconnection, the right piece is open.
theorem disconnection_imp_open_right[T](open: Set[T] -> Bool, u: Set[T], v: Set[T]) {
    is_disconnection[T](open, u, v) implies open(v)
} by {
    if is_disconnection[T](open, u, v) {
        is_disconnection_iff[T](open, u, v)
        if not open(v) {
            false
        }
    }
}

/// From a disconnection, the disjointness/cover/nonemptiness fields.
theorem disconnection_imp_fields[T](open: Set[T] -> Bool, u: Set[T], v: Set[T]) {
    is_disconnection[T](open, u, v) implies
        (u != Set[T].empty_set and v != Set[T].empty_set
            and u.is_disjoint(v) and u.union(v) = Set[T].universal_set)
} by {
    if is_disconnection[T](open, u, v) {
        is_disconnection_iff[T](open, u, v)
        (u != Set[T].empty_set and v != Set[T].empty_set
            and u.is_disjoint(v) and u.union(v) = Set[T].universal_set)
    }
}

/// From a disconnection, the left piece is clopen.
theorem disconnection_left_clopen[T](open: Set[T] -> Bool, u: Set[T], v: Set[T]) {
    is_disconnection[T](open, u, v) implies is_clopen[T](open, u)
} by {
    if is_disconnection[T](open, u, v) {
        disconnection_imp_open_left[T](open, u, v)
        disconnection_imp_open_right[T](open, u, v)
        disconnection_imp_fields[T](open, u, v)
        compl_eq_of_disjoint_cover[T](u, v)
        is_clopen_intro[T](open, u)
        is_clopen[T](open, u)
    }
}

/// A nonempty clopen set is the whole space when only the trivial clopen sets exist.
theorem via_clopen_nonempty_clopen_universal[T](open: Set[T] -> Bool, s: Set[T]) {
    is_connected_via_clopen[T](open) and is_clopen[T](open, s) and s != Set[T].empty_set
        implies s = Set[T].universal_set
} by {
    if is_connected_via_clopen[T](open) and is_clopen[T](open, s) and s != Set[T].empty_set {
        is_connected_via_clopen_iff[T](open)
        if s != Set[T].universal_set {
            false
        }
    }
}

/// A space with only trivial clopen sets is connected.
theorem via_clopen_imp_connected[T](open: Set[T] -> Bool) {
    is_connected_via_clopen[T](open) implies is_connected[T](open)
} by {
    if is_connected_via_clopen[T](open) {
        is_connected_via_clopen_iff[T](open)
        is_connected_iff[T](open)
        if exists(u: Set[T], v: Set[T]) { is_disconnection[T](open, u, v) } {
            let u: Set[T] satisfy {
                exists(v: Set[T]) { is_disconnection[T](open, u, v) }
            }
            let v: Set[T] satisfy {
                is_disconnection[T](open, u, v)
            }
            disconnection_left_clopen[T](open, u, v)
            disconnection_imp_fields[T](open, u, v)
            compl_eq_of_disjoint_cover[T](u, v)
            if u = Set[T].universal_set {
                universal_set_compl_is_empty[T]
                false
            }
            via_clopen_nonempty_clopen_universal[T](open, u)
            false
        }
        not exists(u: Set[T], v: Set[T]) {
            is_disconnection[T](open, u, v)
        }
    }
}
