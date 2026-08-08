from data.basic.set import Set, set_preimage, set_preimage_contains_image
from analysis.topology.topological_space import TopologicalSpace, is_continuous, Subspace,
    subspace_open, has_subspace_witness

/// The inclusion map sending a point of the subspace `a` to its ambient value.
define subspace_inclusion[T: TopologicalSpace](a: Set[T], p: Subspace[T, a]) -> T {
    p.value
}

/// Membership of a point in the preimage of `v` under the inclusion coincides with
/// membership of its ambient value in `v`.
theorem subspace_inclusion_preimage_contains[T: TopologicalSpace](
    a: Set[T], v: Set[T], p: Subspace[T, a]
) {
    set_preimage(subspace_inclusion[T](a), v).contains(p) = v.contains(p.value)
} by {
    let f = subspace_inclusion[T](a)
    f(p) = p.value
    set_preimage_contains_image(f, v, p)
    set_preimage(f, v).contains(p) = v.contains(f(p))
}

/// The inclusion of a subspace into its ambient space is continuous.
theorem subspace_inclusion_continuous[T: TopologicalSpace](a: Set[T]) {
    is_continuous(subspace_inclusion[T](a))
} by {
    let f = subspace_inclusion[T](a)
    forall(v: Set[T]) {
        if T.is_open(v) {
            let pre = set_preimage(f, v)
            forall(p: Subspace[T, a]) {
                subspace_inclusion_preimage_contains[T](a, v, p)
                pre.contains(p) = v.contains(p.value)
            }
            T.is_open(v) and forall(p: Subspace[T, a]) {
                pre.contains(p) = v.contains(p.value)
            }
            has_subspace_witness(a, pre, v)
            subspace_open(a, pre)
            Subspace[T, a].is_open(pre)
            Subspace[T, a].is_open(set_preimage(f, v))
        }
    }
    is_continuous(f) = forall(v: Set[T]) {
        T.is_open(v) implies Subspace[T, a].is_open(set_preimage(f, v))
    }
    forall(v: Set[T]) {
        T.is_open(v) implies Subspace[T, a].is_open(set_preimage(f, v))
    }
    is_continuous(f)
}
