/// Connected subsets of a topological space.
///
/// A subset is connected when it cannot be split into two disjoint nonempty
/// relatively-open subsets.  On the real line, connected subsets are exactly
/// the intervals: an order-convex set (containing every point between any two
/// of its points) is connected, and a connected set is order-convex.  In
/// particular, every closed interval `[lower, upper]` is connected.  The
/// continuous image of a connected set is connected.

from list import List
from data.basic.set import Set, set_ext, compl_contains_eq, intersection_contains_eq,
    intersection_contains_intro, intersection_comm, union_contains_eq, union_comm, subset_contains,
    subset_trans, subset_antisymm, empty_set_contains_eq, universal_set_contains_eq,
    set_preimage, set_preimage_contains_eq, set_preimage_intersection,
    set_image, maps_into_set_image, set_image_contains_witness
from order import lt_of_lt_of_lte, lt_imp_lte, not_lt_self, lt_or_lte, min_lte_left,
    min_lte_right, lt_min_iff, closed_interval
from order_set import closed_interval_set, closed_interval_set_contains_eq
from real import Real, lte_self, lte_trans, lt_trans, lt_lte_trans, lte_lt_trans,
    lte_or_gte, lte_both_ways_imp_eq, not_lt_imp_gte, bounds_imp_close,
    lt_imp_minus_pos, eps_lt_half, lt_add_pos, lt_add_right, sub_cancels,
    is_nonempty, is_set_upper_bound, has_upper_bound, is_set_supremum, completeness
from analysis.topology.topological_space import TopologicalSpace, is_open_in_subspace,
    subspace_open_from_ambient, is_continuous
from analysis.topology.continuous_preimage import continuous_open_preimage
from analysis.topology.borel_functions import real_is_open, real_is_interior_point,
    real_open_is_interior_point
from analysis.topology.real_topology import open_lower_ray_set, open_upper_ray_set,
    open_lower_ray_set_contains_eq, open_upper_ray_set_contains_eq,
    real_open_lower_ray_set, real_open_upper_ray_set, positive_half_lt_bound,
    sub_lt_self_of_pos, supremum_approximation, nonempty_from_neq_empty
from data.basic.logic import by_contradiction, not_forall_imp_exists_not, not_implies

/// Membership in a closed interval gives the lower endpoint inequality.
theorem closed_interval_left_local(a: Real, b: Real, x: Real) {
    closed_interval(a, b, x) implies a <= x
} by {
    if closed_interval(a, b, x) {
        closed_interval(a, b, x) = (a <= x and x <= b)
        a <= x and x <= b
        a <= x
    }
}

/// Membership in a closed interval gives the upper endpoint inequality.
theorem closed_interval_right_local(a: Real, b: Real, x: Real) {
    closed_interval(a, b, x) implies x <= b
} by {
    if closed_interval(a, b, x) {
        closed_interval(a, b, x) = (a <= x and x <= b)
        a <= x and x <= b
        x <= b
    }
}

/// True if a subset is connected in the subspace topology: it cannot be written as
/// the union of two disjoint nonempty relatively-open subsets.
define is_connected_set[T: TopologicalSpace](s: Set[T]) -> Bool {
    not exists(u: Set[T], v: Set[T]) {
        is_open_in_subspace(s, u) and is_open_in_subspace(s, v) and u != Set[T].empty_set and v != Set[T].empty_set and u.is_disjoint(v) and u.union(v) = s
    }
}

/// Unfolding lemma for `is_connected_set`.
theorem is_connected_set_iff[T: TopologicalSpace](s: Set[T]) {
    is_connected_set(s) = not exists(u: Set[T], v: Set[T]) {
        is_open_in_subspace(s, u) and is_open_in_subspace(s, v) and u != Set[T].empty_set and v != Set[T].empty_set and u.is_disjoint(v) and u.union(v) = s
    }
} by {
}

/// Two relatively-open sets of a convex real set, one containing `x` and the other
/// containing `y` with `x < y`, cannot cover the set disjointly.
///
/// This is the interval version of the classical supremum argument: if they did,
/// the supremum of `u ∩ [x, y]` would lie in `u ∪ v` and, by relative openness,
/// would be flanked by points of the opposite side, contradicting the supremum
/// property.
theorem no_relative_disconnection_with_order(
    a: Set[Real], u: Set[Real], v: Set[Real], x: Real, y: Real) {
    not (is_open_in_subspace[Real](a, u) and is_open_in_subspace[Real](a, v) and u.is_disjoint(v) and u.union(v) = a and u.contains(x) and v.contains(y) and x < y and closed_interval_set(x, y).subset(a))
} by {
    if is_open_in_subspace[Real](a, u) and is_open_in_subspace[Real](a, v) and u.is_disjoint(v) and u.union(v) = a and u.contains(x) and v.contains(y) and x < y and closed_interval_set(x, y).subset(a) {
        is_open_in_subspace[Real](a, u) = exists(w: Set[Real]) {
            Real.is_open(w) and u = w.intersection(a)
        }
        let u_wit: Set[Real] satisfy {
            Real.is_open(u_wit) and u = u_wit.intersection(a)
        }
        is_open_in_subspace[Real](a, v) = exists(w: Set[Real]) {
            Real.is_open(w) and v = w.intersection(a)
        }
        let v_wit: Set[Real] satisfy {
            Real.is_open(v_wit) and v = v_wit.intersection(a)
        }
        u.is_disjoint(v)
        u.union(v) = a
        u.contains(x)
        v.contains(y)
        x < y
        closed_interval_set(x, y).subset(a)
        let s = u.intersection(closed_interval_set(x, y))
        lte_self(x)
        x <= x
        lt_imp_lte(x, y)
        x <= y
        closed_interval(x, y, x)
        closed_interval_set_contains_eq(x, y, x)
        closed_interval_set(x, y).contains(x)
        intersection_contains_intro(u, closed_interval_set(x, y), x)
        s.contains(x)
        is_nonempty(s)
        forall(z: Real) {
            if s.contains(z) {
                intersection_contains_eq(u, closed_interval_set(x, y), z)
                closed_interval_set(x, y).contains(z)
                closed_interval_set_contains_eq(x, y, z)
                closed_interval(x, y, z)
                z <= y
            }
        }
        is_set_upper_bound(s, y)
        has_upper_bound(s)
        completeness(s)
        let c: Real satisfy {
            is_set_supremum(s, c)
        }
        is_set_supremum(s, c)
        is_set_upper_bound(s, c)
        is_set_supremum(s, c) = (is_set_upper_bound(s, c) and forall(b: Real) { is_set_upper_bound(s, b) implies c <= b })
        forall(b: Real) { is_set_upper_bound(s, b) implies c <= b }
        is_set_upper_bound(s, y) implies c <= y
        c <= y
        is_set_upper_bound(s, c) = forall(z: Real) { s.contains(z) implies z <= c }
        s.contains(x) implies x <= c
        x <= c
        closed_interval(x, y, c)
        closed_interval_set_contains_eq(x, y, c)
        closed_interval_set(x, y).contains(c)
        subset_contains(closed_interval_set(x, y), a, c)
        a.contains(c)
        u.union(v) = a
        u.union(v).contains(c)
        union_contains_eq(u, v, c)
        u.contains(c) or v.contains(c)
        if u.contains(c) {
            u = u_wit.intersection(a)
            u_wit.intersection(a).contains(c)
            intersection_contains_eq(u_wit, a, c)
            u_wit.contains(c)
            real_is_open(u_wit)
            real_open_is_interior_point(u_wit, c)
            real_is_interior_point(u_wit, c)
            real_is_interior_point(u_wit, c) = (u_wit.contains(c) and exists(eps: Real) {
                eps.is_positive and forall(w: Real) { w.is_close(c, eps) implies u_wit.contains(w) }
            })
            exists(eps: Real) {
                eps.is_positive and forall(w: Real) { w.is_close(c, eps) implies u_wit.contains(w) }
            }
            let eps0: Real satisfy {
                eps0.is_positive and forall(w: Real) { w.is_close(c, eps0) implies u_wit.contains(w) }
            }
            eps0.is_positive
            if c = y {
                u.is_disjoint(v)
                not (u.contains(c) and v.contains(c))
                u.contains(c)
                v.contains(c)
                false
            }
            c != y
            if not (c < y) {
                not_lt_imp_gte(c, y)
                c >= y
                lte_both_ways_imp_eq(c, y)
                c = y
                false
            }
            c < y
            lt_imp_minus_pos(c, y)
            (y - c).is_positive
            eps_lt_half(eps0)
            let d0: Real satisfy {
                d0.is_positive and d0 + d0 < eps0
            }
            d0.is_positive
            positive_half_lt_bound(d0, eps0)
            d0 < eps0
            eps_lt_half(y - c)
            let d1: Real satisfy {
                d1.is_positive and d1 + d1 < y - c
            }
            d1.is_positive
            positive_half_lt_bound(d1, y - c)
            d1 < y - c
            let delta = d0.min(d1)
            lt_min_iff(Real.0, d0, d1)
            Real.0 < d0.min(d1) = (Real.0 < d0 and Real.0 < d1)
            Real.0 < d0
            Real.0 < d1
            Real.0 < d0.min(d1)
            Real.0 < delta
            min_lte_left(d0, d1)
            d0.min(d1) <= d0
            delta <= d0
            lte_lt_trans(delta, d0, eps0)
            delta < eps0
            min_lte_right(d0, d1)
            d0.min(d1) <= d1
            delta <= d1
            lte_lt_trans(delta, d1, y - c)
            delta < y - c
            lt_add_pos(c, delta)
            c < c + delta
            lt_add_right(delta, eps0, c)
            c + delta < c + eps0
            lt_add_right(delta, y - c, c)
            c + delta < c + (y - c)
            sub_cancels(y, c)
            y + c - c = y
            c + (y - c) = y
            c + delta < y
            sub_lt_self_of_pos(c, eps0)
            c - eps0 < c
            lt_trans(c - eps0, c, c + delta)
            c - eps0 < c + delta
            bounds_imp_close(c + delta, c, eps0)
            (c + delta).is_close(c, eps0)
            u_wit.contains(c + delta)
            lt_imp_lte(c + delta, y)
            c + delta <= y
            lte_lt_trans(x, c, c + delta)
            x < c + delta
            lt_imp_lte(x, c + delta)
            x <= c + delta
            closed_interval(x, y, c + delta)
            closed_interval_set_contains_eq(x, y, c + delta)
            closed_interval_set(x, y).contains(c + delta)
            subset_contains(closed_interval_set(x, y), a, c + delta)
            a.contains(c + delta)
            intersection_contains_intro(u_wit, a, c + delta)
            u_wit.intersection(a).contains(c + delta)
            u.contains(c + delta)
            intersection_contains_intro(u, closed_interval_set(x, y), c + delta)
            s.contains(c + delta)
            is_set_upper_bound(s, c) = forall(z: Real) { s.contains(z) implies z <= c }
            s.contains(c + delta) implies (c + delta) <= c
            c + delta <= c
            lt_of_lt_of_lte(c, c + delta, c)
            c < c
            not_lt_self(c)
            false
        } else {
            v.contains(c)
            v = v_wit.intersection(a)
            v_wit.intersection(a).contains(c)
            intersection_contains_eq(v_wit, a, c)
            v_wit.contains(c)
            real_is_open(v_wit)
            real_open_is_interior_point(v_wit, c)
            real_is_interior_point(v_wit, c)
            real_is_interior_point(v_wit, c) = (v_wit.contains(c) and exists(eps: Real) {
                eps.is_positive and forall(w: Real) { w.is_close(c, eps) implies v_wit.contains(w) }
            })
            exists(eps: Real) {
                eps.is_positive and forall(w: Real) { w.is_close(c, eps) implies v_wit.contains(w) }
            }
            let eps1: Real satisfy {
                eps1.is_positive and forall(w: Real) { w.is_close(c, eps1) implies v_wit.contains(w) }
            }
            eps1.is_positive
            supremum_approximation(s, c, eps1)
            let z: Real satisfy {
                s.contains(z) and c - eps1 < z
            }
            s.contains(z)
            c - eps1 < z
            intersection_contains_eq(u, closed_interval_set(x, y), z)
            u.contains(z)
            is_set_upper_bound(s, c) = forall(w: Real) { s.contains(w) implies w <= c }
            s.contains(z) implies z <= c
            z <= c
            lt_add_pos(c, eps1)
            c < c + eps1
            lte_lt_trans(z, c, c + eps1)
            z < c + eps1
            bounds_imp_close(z, c, eps1)
            z.is_close(c, eps1)
            v_wit.contains(z)
            intersection_contains_eq(u, closed_interval_set(x, y), z)
            closed_interval_set(x, y).contains(z)
            subset_contains(closed_interval_set(x, y), a, z)
            a.contains(z)
            intersection_contains_intro(v_wit, a, z)
            v_wit.intersection(a).contains(z)
            v.contains(z)
            u.is_disjoint(v)
            not (u.contains(z) and v.contains(z))
            u.contains(z) and v.contains(z)
            false
        }
    }
    not (is_open_in_subspace[Real](a, u) and is_open_in_subspace[Real](a, v) and u.is_disjoint(v) and u.union(v) = a and u.contains(x) and v.contains(y) and x < y and closed_interval_set(x, y).subset(a))
}

/// A set unequal to the empty set has a member.
theorem nonempty_from_neq_empty_generic[K](t: Set[K]) {
    t != Set[K].empty_set implies exists(x: K) { t.contains(x) }
} by {
    if t != Set[K].empty_set {
        if not exists(x: K) { t.contains(x) } {
            forall(x: K) {
                if t.contains(x) {
                    not exists(w: K) { t.contains(w) }
                    false
                }
                not t.contains(x)
                empty_set_contains_eq[K](x)
                t.contains(x) = Set[K].empty_set.contains(x)
            }
            set_ext(t, Set[K].empty_set)
            t = Set[K].empty_set
            false
        }
        exists(x: K) { t.contains(x) }
    }
}

/// A set containing a point is nonempty.
theorem nonempty_of_contains[K](t: Set[K], x: K) {
    t.contains(x) implies t != Set[K].empty_set
} by {
    if t.contains(x) {
        if t = Set[K].empty_set {
            empty_set_contains_eq[K](x)
            not Set[K].empty_set.contains(x)
            false
        }
        t != Set[K].empty_set
    }
}

/// True if a set of reals is order-convex: it contains every point between any
/// two of its points.
define is_order_convex_set(a: Set[Real]) -> Bool {
    forall(x: Real, y: Real, z: Real) {
        a.contains(x) and a.contains(y) and x <= z and z <= y implies a.contains(z)
    }
}

/// Unfolding lemma for `is_order_convex_set`.
theorem is_order_convex_set_iff(a: Set[Real]) {
    is_order_convex_set(a) = forall(x: Real, y: Real, z: Real) {
        a.contains(x) and a.contains(y) and x <= z and z <= y implies a.contains(z)
    }
} by {
}

/// Every closed interval is order-convex.
theorem closed_interval_set_is_order_convex(lower: Real, upper: Real) {
    is_order_convex_set(closed_interval_set(lower, upper))
} by {
    if not is_order_convex_set(closed_interval_set(lower, upper)) {
        is_order_convex_set_iff(closed_interval_set(lower, upper))
        is_order_convex_set(closed_interval_set(lower, upper)) = forall(x: Real, y: Real, z: Real) {
            (closed_interval_set(lower, upper)).contains(x) and (closed_interval_set(lower, upper)).contains(y) and x <= z and z <= y implies (closed_interval_set(lower, upper)).contains(z)
        }
        not forall(x: Real, y: Real, z: Real) {
            (closed_interval_set(lower, upper)).contains(x) and (closed_interval_set(lower, upper)).contains(y) and x <= z and z <= y implies (closed_interval_set(lower, upper)).contains(z)
        }
        not_forall_imp_exists_not[Real](function(x: Real) {
            forall(y: Real, z: Real) {
                (closed_interval_set(lower, upper)).contains(x) and (closed_interval_set(lower, upper)).contains(y) and x <= z and z <= y implies (closed_interval_set(lower, upper)).contains(z)
            }
        })
        let x: Real satisfy {
            not forall(y: Real, z: Real) {
                (closed_interval_set(lower, upper)).contains(x) and (closed_interval_set(lower, upper)).contains(y) and x <= z and z <= y implies (closed_interval_set(lower, upper)).contains(z)
            }
        }
        not_forall_imp_exists_not[Real](function(y: Real) {
            forall(z: Real) {
                (closed_interval_set(lower, upper)).contains(x) and (closed_interval_set(lower, upper)).contains(y) and x <= z and z <= y implies (closed_interval_set(lower, upper)).contains(z)
            }
        })
        let y: Real satisfy {
            not forall(z: Real) {
                (closed_interval_set(lower, upper)).contains(x) and (closed_interval_set(lower, upper)).contains(y) and x <= z and z <= y implies (closed_interval_set(lower, upper)).contains(z)
            }
        }
        not_forall_imp_exists_not[Real](function(z: Real) {
            (closed_interval_set(lower, upper)).contains(x) and (closed_interval_set(lower, upper)).contains(y) and x <= z and z <= y implies (closed_interval_set(lower, upper)).contains(z)
        })
        let z: Real satisfy {
            not ((closed_interval_set(lower, upper)).contains(x) and (closed_interval_set(lower, upper)).contains(y) and x <= z and z <= y implies (closed_interval_set(lower, upper)).contains(z))
        }
        not_implies((closed_interval_set(lower, upper)).contains(x) and (closed_interval_set(lower, upper)).contains(y) and x <= z and z <= y, (closed_interval_set(lower, upper)).contains(z))
        (not ((closed_interval_set(lower, upper)).contains(x) and (closed_interval_set(lower, upper)).contains(y) and x <= z and z <= y implies (closed_interval_set(lower, upper)).contains(z))) = ((closed_interval_set(lower, upper)).contains(x) and (closed_interval_set(lower, upper)).contains(y) and x <= z and z <= y and not (closed_interval_set(lower, upper)).contains(z))
        ((closed_interval_set(lower, upper)).contains(x) and (closed_interval_set(lower, upper)).contains(y) and x <= z and z <= y and not (closed_interval_set(lower, upper)).contains(z))
        ((closed_interval_set(lower, upper)).contains(x) and (closed_interval_set(lower, upper)).contains(y) and x <= z and z <= y)
        (closed_interval_set(lower, upper)).contains(x)
        (closed_interval_set(lower, upper)).contains(y)
        x <= z
        z <= y
        closed_interval_set_contains_eq(lower, upper, x)
        closed_interval(lower, upper, x)
        closed_interval_set_contains_eq(lower, upper, y)
        closed_interval(lower, upper, y)
        lte_trans(lower, x, z)
        lower <= z
        lte_trans(z, y, upper)
        z <= upper
        closed_interval(lower, upper, z)
        closed_interval_set_contains_eq(lower, upper, z)
        (closed_interval_set(lower, upper)).contains(z)
        not (closed_interval_set(lower, upper)).contains(z)
        false
    }
    by_contradiction(is_order_convex_set(closed_interval_set(lower, upper)))
    (not is_order_convex_set(closed_interval_set(lower, upper)) implies false) implies is_order_convex_set(closed_interval_set(lower, upper))
    is_order_convex_set(closed_interval_set(lower, upper))
}

/// An order-convex set of reals is connected: a relative disconnection of it
/// would separate two points `x < y` of it, contradicting convexity via the
/// supremum argument.
theorem order_convex_imp_connected_set(a: Set[Real]) {
    is_order_convex_set(a) implies is_connected_set[Real](a)
} by {
    if is_order_convex_set(a) {
        is_connected_set_iff[Real](a)
        if exists(u: Set[Real], v: Set[Real]) {
            is_open_in_subspace[Real](a, u) and is_open_in_subspace[Real](a, v) and u != Set[Real].empty_set and v != Set[Real].empty_set and u.is_disjoint(v) and u.union(v) = a
        } {
            let u: Set[Real] satisfy {
                exists(v: Set[Real]) {
                    is_open_in_subspace[Real](a, u) and is_open_in_subspace[Real](a, v) and u != Set[Real].empty_set and v != Set[Real].empty_set and u.is_disjoint(v) and u.union(v) = a
                }
            }
            let v: Set[Real] satisfy {
                is_open_in_subspace[Real](a, u) and is_open_in_subspace[Real](a, v) and u != Set[Real].empty_set and v != Set[Real].empty_set and u.is_disjoint(v) and u.union(v) = a
            }
            u != Set[Real].empty_set
            nonempty_from_neq_empty_generic[Real](u)
            let x: Real satisfy {
                u.contains(x)
            }
            v != Set[Real].empty_set
            nonempty_from_neq_empty_generic[Real](v)
            let y: Real satisfy {
                v.contains(y)
            }
            u.contains(x)
            v.contains(y)
            u.is_disjoint(v)
            u.union(v) = a
            is_open_in_subspace[Real](a, u) = exists(w: Set[Real]) {
                Real.is_open(w) and u = w.intersection(a)
            }
            let u_wit: Set[Real] satisfy {
                Real.is_open(u_wit) and u = u_wit.intersection(a)
            }
            u = u_wit.intersection(a)
            u_wit.intersection(a).contains(x)
            intersection_contains_eq(u_wit, a, x)
            a.contains(x)
            is_open_in_subspace[Real](a, v) = exists(w: Set[Real]) {
                Real.is_open(w) and v = w.intersection(a)
            }
            let v_wit: Set[Real] satisfy {
                Real.is_open(v_wit) and v = v_wit.intersection(a)
            }
            v = v_wit.intersection(a)
            v_wit.intersection(a).contains(y)
            intersection_contains_eq(v_wit, a, y)
            a.contains(y)
            if x = y {
                u.is_disjoint(v)
                not (u.contains(x) and v.contains(x))
                v.contains(x)
                u.contains(x) and v.contains(x)
                false
            }
            x != y
            lt_or_lte(x, y)
            if x < y {
                forall(w: Real) {
                    if closed_interval_set(x, y).contains(w) {
                        closed_interval_set_contains_eq(x, y, w)
                        closed_interval(x, y, w)
                        closed_interval_left_local(x, y, w)
                        x <= w
                        closed_interval_right_local(x, y, w)
                        w <= y
                        is_order_convex_set_iff(a)
                        forall(x1: Real, y1: Real, z1: Real) {
                            a.contains(x1) and a.contains(y1) and x1 <= z1 and z1 <= y1 implies a.contains(z1)
                        }
                        x <= w
                        w <= y
                        x <= w and w <= y
                        a.contains(y) and x <= w and w <= y
                        a.contains(x) and a.contains(y) and x <= w and w <= y
                        a.contains(x) and a.contains(y) and x <= w and w <= y implies a.contains(w)
                        a.contains(w)
                    }
                }
                closed_interval_set(x, y).subset(a)
                no_relative_disconnection_with_order(a, u, v, x, y)
                false
            }
            lt_or_lte(y, x)
            if y < x {
                forall(w: Real) {
                    if v.contains(w) and u.contains(w) {
                        u.is_disjoint(v)
                        not (u.contains(w) and v.contains(w))
                        false
                    }
                    not (v.contains(w) and u.contains(w))
                }
                v.is_disjoint(u)
                union_comm(u, v)
                u.union(v) = v.union(u)
                v.union(u) = a
                forall(w: Real) {
                    if closed_interval_set(y, x).contains(w) {
                        closed_interval_set_contains_eq(y, x, w)
                        closed_interval(y, x, w)
                        closed_interval_left_local(y, x, w)
                        y <= w
                        closed_interval_right_local(y, x, w)
                        w <= x
                        is_order_convex_set_iff(a)
                        forall(x1: Real, y1: Real, z1: Real) {
                            a.contains(x1) and a.contains(y1) and x1 <= z1 and z1 <= y1 implies a.contains(z1)
                        }
                        y <= w
                        w <= x
                        y <= w and w <= x
                        a.contains(x) and y <= w and w <= x
                        a.contains(y) and a.contains(x) and y <= w and w <= x
                        a.contains(y) and a.contains(x) and y <= w and w <= x implies a.contains(w)
                        a.contains(w)
                    }
                }
                closed_interval_set(y, x).subset(a)
                no_relative_disconnection_with_order(a, v, u, y, x)
                false
            }
            x <= y
            lte_both_ways_imp_eq(x, y)
            x = y
            false
        }
        not exists(u: Set[Real], v: Set[Real]) {
            is_open_in_subspace[Real](a, u) and is_open_in_subspace[Real](a, v) and u != Set[Real].empty_set and v != Set[Real].empty_set and u.is_disjoint(v) and u.union(v) = a
        }
        is_connected_set[Real](a)
    }
}

/// Every closed interval of reals is connected.
theorem closed_interval_is_connected_set(lower: Real, upper: Real) {
    is_connected_set[Real](closed_interval_set(lower, upper))
} by {
    closed_interval_set_is_order_convex(lower, upper)
    order_convex_imp_connected_set(closed_interval_set(lower, upper))
    is_connected_set[Real](closed_interval_set(lower, upper))
}

/// A connected set of reals is order-convex, in pointwise form.
theorem connected_set_order_convex_forall(a: Set[Real]) {
    is_connected_set[Real](a) implies forall(x: Real, y: Real, z: Real) {
        a.contains(x) and a.contains(y) and x <= z and z <= y implies a.contains(z)
    }
} by {
    if is_connected_set[Real](a) {
        is_connected_set_iff[Real](a)
        forall(x: Real, y: Real, z: Real) {
            if a.contains(x) and a.contains(y) and x <= z and z <= y {
                if not a.contains(z) {
                    let u: Set[Real] = a.intersection(open_lower_ray_set(z))
                    let v: Set[Real] = a.intersection(open_upper_ray_set(z))
                    real_open_lower_ray_set(z)
                    real_is_open(open_lower_ray_set(z))
                    Real.is_open(open_lower_ray_set(z))
                    subspace_open_from_ambient[Real](a, open_lower_ray_set(z))
                    is_open_in_subspace[Real](a, open_lower_ray_set(z).intersection(a))
                    intersection_comm(open_lower_ray_set(z), a)
                    open_lower_ray_set(z).intersection(a) = a.intersection(open_lower_ray_set(z))
                    is_open_in_subspace[Real](a, u)
                    real_open_upper_ray_set(z)
                    real_is_open(open_upper_ray_set(z))
                    Real.is_open(open_upper_ray_set(z))
                    subspace_open_from_ambient[Real](a, open_upper_ray_set(z))
                    is_open_in_subspace[Real](a, open_upper_ray_set(z).intersection(a))
                    intersection_comm(open_upper_ray_set(z), a)
                    open_upper_ray_set(z).intersection(a) = a.intersection(open_upper_ray_set(z))
                    is_open_in_subspace[Real](a, v)
                    // x < z
                    if not (x < z) {
                        not_lt_imp_gte(x, z)
                        x >= z
                        z <= x
                        lte_both_ways_imp_eq(x, z)
                        x = z
                        a.contains(z)
                        not a.contains(z)
                        false
                    }
                    x < z
                    open_lower_ray_set_contains_eq(z, x)
                    open_lower_ray_set(z).contains(x)
                    a.contains(x)
                    intersection_contains_intro(a, open_lower_ray_set(z), x)
                    a.intersection(open_lower_ray_set(z)).contains(x)
                    intersection_comm(a, open_lower_ray_set(z))
                    a.intersection(open_lower_ray_set(z)) = open_lower_ray_set(z).intersection(a)
                    u.contains(x)
                    nonempty_of_contains(u, x)
                    u != Set[Real].empty_set
                    // z < y
                    if not (z < y) {
                        not_lt_imp_gte(z, y)
                        z >= y
                        y <= z
                        lte_both_ways_imp_eq(z, y)
                        z = y
                        a.contains(z)
                        not a.contains(z)
                        false
                    }
                    z < y
                    open_upper_ray_set_contains_eq(z, y)
                    open_upper_ray_set(z).contains(y)
                    a.contains(y)
                    intersection_contains_intro(a, open_upper_ray_set(z), y)
                    a.intersection(open_upper_ray_set(z)).contains(y)
                    intersection_comm(a, open_upper_ray_set(z))
                    a.intersection(open_upper_ray_set(z)) = open_upper_ray_set(z).intersection(a)
                    v.contains(y)
                    nonempty_of_contains(v, y)
                    v != Set[Real].empty_set
                    // disjoint
                    u.is_disjoint(v) = forall(w: Real) {
                        not (u.contains(w) and v.contains(w))
                    }
                    forall(w: Real) {
                        if u.contains(w) and v.contains(w) {
                            intersection_comm(a, open_lower_ray_set(z))
                            a.intersection(open_lower_ray_set(z)) = open_lower_ray_set(z).intersection(a)
                            u = open_lower_ray_set(z).intersection(a)
                            intersection_comm(a, open_upper_ray_set(z))
                            a.intersection(open_upper_ray_set(z)) = open_upper_ray_set(z).intersection(a)
                            v = open_upper_ray_set(z).intersection(a)
                            intersection_contains_eq(open_lower_ray_set(z), a, w)
                            open_lower_ray_set(z).contains(w)
                            open_lower_ray_set_contains_eq(z, w)
                            open_lower_ray_set(z).contains(w) = (w < z)
                            w < z
                            intersection_contains_eq(open_upper_ray_set(z), a, w)
                            open_upper_ray_set(z).contains(w)
                            open_upper_ray_set_contains_eq(z, w)
                            open_upper_ray_set(z).contains(w) = (z < w)
                            z < w
                            lt_trans(w, z, w)
                            w < w
                            not_lt_self(w)
                            false
                        }
                        not (u.contains(w) and v.contains(w))
                    }
                    u.is_disjoint(v)
                    // cover
                    forall(w: Real) {
                        if a.contains(w) {
                            if w = z {
                                a.contains(z)
                                not a.contains(z)
                                false
                            }
                            w != z
                            lt_or_lte(w, z)
                            if w < z {
                                open_lower_ray_set_contains_eq(z, w)
                                open_lower_ray_set(z).contains(w)
                                a.contains(w)
                                intersection_contains_intro(a, open_lower_ray_set(z), w)
                                a.intersection(open_lower_ray_set(z)).contains(w)
                                intersection_comm(a, open_lower_ray_set(z))
                                u.contains(w)
                                union_contains_eq(u, v, w)
                                u.union(v).contains(w)
                            } else {
                                z <= w
                                if not (z < w) {
                                    not_lt_imp_gte(z, w)
                                    z >= w
                                    w <= z
                                    lte_both_ways_imp_eq(z, w)
                                    z = w
                                    w = z
                                    a.contains(z)
                                    not a.contains(z)
                                    false
                                }
                                z < w
                                open_upper_ray_set_contains_eq(z, w)
                                open_upper_ray_set(z).contains(w)
                                a.contains(w)
                                intersection_contains_intro(a, open_upper_ray_set(z), w)
                                a.intersection(open_upper_ray_set(z)).contains(w)
                                intersection_comm(a, open_upper_ray_set(z))
                                v.contains(w)
                                union_contains_eq(u, v, w)
                                u.union(v).contains(w)
                            }
                        }
                    }
                    a.subset(u.union(v))
                    forall(w: Real) {
                        if u.union(v).contains(w) {
                            union_contains_eq(u, v, w)
                            if u.contains(w) {
                                intersection_contains_eq(a, open_lower_ray_set(z), w)
                                a.contains(w)
                            } else {
                                v.contains(w)
                                intersection_contains_eq(a, open_upper_ray_set(z), w)
                                a.contains(w)
                            }
                        }
                    }
                    u.union(v).subset(a)
                    subset_antisymm(a, u.union(v))
                    u.union(v) = a
                    exists(uu: Set[Real], vv: Set[Real]) {
                        is_open_in_subspace[Real](a, uu) and is_open_in_subspace[Real](a, vv) and uu != Set[Real].empty_set and vv != Set[Real].empty_set and uu.is_disjoint(vv) and uu.union(vv) = a
                    }
                    not exists(uu: Set[Real], vv: Set[Real]) {
                        is_open_in_subspace[Real](a, uu) and is_open_in_subspace[Real](a, vv) and uu != Set[Real].empty_set and vv != Set[Real].empty_set and uu.is_disjoint(vv) and uu.union(vv) = a
                    }
                    false
                }
                a.contains(z)
            }
        }
    }
}

/// A connected set of reals is order-convex: it contains every point between
/// any two of its points.
theorem connected_set_imp_order_convex(a: Set[Real]) {
    is_connected_set[Real](a) implies is_order_convex_set(a)
} by {
    if is_connected_set[Real](a) {
        if not is_order_convex_set(a) {
            is_order_convex_set_iff(a)
            is_order_convex_set(a) = forall(x: Real, y: Real, z: Real) {
                a.contains(x) and a.contains(y) and x <= z and z <= y implies a.contains(z)
            }
            not forall(x: Real, y: Real, z: Real) {
                a.contains(x) and a.contains(y) and x <= z and z <= y implies a.contains(z)
            }
            not_forall_imp_exists_not[Real](function(x: Real) {
                forall(y: Real, z: Real) {
                    a.contains(x) and a.contains(y) and x <= z and z <= y implies a.contains(z)
                }
            })
            let x: Real satisfy {
                not forall(y: Real, z: Real) {
                    a.contains(x) and a.contains(y) and x <= z and z <= y implies a.contains(z)
                }
            }
            not_forall_imp_exists_not[Real](function(y: Real) {
                forall(z: Real) {
                    a.contains(x) and a.contains(y) and x <= z and z <= y implies a.contains(z)
                }
            })
            let y: Real satisfy {
                not forall(z: Real) {
                    a.contains(x) and a.contains(y) and x <= z and z <= y implies a.contains(z)
                }
            }
            not_forall_imp_exists_not[Real](function(z: Real) {
                a.contains(x) and a.contains(y) and x <= z and z <= y implies a.contains(z)
            })
            let z: Real satisfy {
                not (a.contains(x) and a.contains(y) and x <= z and z <= y implies a.contains(z))
            }
            not_implies(a.contains(x) and a.contains(y) and x <= z and z <= y, a.contains(z))
            (not (a.contains(x) and a.contains(y) and x <= z and z <= y implies a.contains(z))) = (a.contains(x) and a.contains(y) and x <= z and z <= y and not a.contains(z))
            (a.contains(x) and a.contains(y) and x <= z and z <= y and not a.contains(z))
            (a.contains(x) and a.contains(y) and x <= z and z <= y)
            a.contains(x)
            a.contains(y)
            x <= z
            z <= y
            connected_set_order_convex_forall(a)
            a.contains(x) and a.contains(y) and x <= z and z <= y implies a.contains(z)
            a.contains(z)
            not a.contains(z)
            false
        }
        by_contradiction(is_order_convex_set(a))
        (not is_order_convex_set(a) implies false) implies is_order_convex_set(a)
        is_order_convex_set(a)
    }
}
/// The connected subsets of the real line are exactly the order-convex sets.
theorem connected_set_iff_order_convex(a: Set[Real]) {
    is_connected_set[Real](a) = is_order_convex_set(a)
} by {
    if is_connected_set[Real](a) {
        connected_set_imp_order_convex(a)
        is_order_convex_set(a)
    }
    if is_order_convex_set(a) {
        order_convex_imp_connected_set(a)
        is_connected_set[Real](a)
    }
    is_connected_set[Real](a) = is_order_convex_set(a)
}

/// The continuous image of a connected set is connected.
theorem continuous_image_connected_set[X: TopologicalSpace, Y: TopologicalSpace](
    f: X -> Y, s: Set[X]) {
    is_continuous(f) and is_connected_set[X](s) implies is_connected_set[Y](set_image(s, f))
} by {
    if is_continuous(f) and is_connected_set[X](s) {
        is_connected_set_iff[Y](set_image(s, f))
        if exists(u: Set[Y], v: Set[Y]) {
            is_open_in_subspace[Y](set_image(s, f), u) and is_open_in_subspace[Y](set_image(s, f), v) and u != Set[Y].empty_set and v != Set[Y].empty_set and u.is_disjoint(v) and u.union(v) = set_image(s, f)
        } {
            let u: Set[Y] satisfy {
                exists(v: Set[Y]) {
                    is_open_in_subspace[Y](set_image(s, f), u) and is_open_in_subspace[Y](set_image(s, f), v) and u != Set[Y].empty_set and v != Set[Y].empty_set and u.is_disjoint(v) and u.union(v) = set_image(s, f)
                }
            }
            let v: Set[Y] satisfy {
                is_open_in_subspace[Y](set_image(s, f), u) and is_open_in_subspace[Y](set_image(s, f), v) and u != Set[Y].empty_set and v != Set[Y].empty_set and u.is_disjoint(v) and u.union(v) = set_image(s, f)
            }
            is_open_in_subspace[Y](set_image(s, f), u) = exists(w: Set[Y]) {
                Y.is_open(w) and u = w.intersection(set_image(s, f))
            }
            let u_wit: Set[Y] satisfy {
                Y.is_open(u_wit) and u = u_wit.intersection(set_image(s, f))
            }
            is_open_in_subspace[Y](set_image(s, f), v) = exists(w: Set[Y]) {
                Y.is_open(w) and v = w.intersection(set_image(s, f))
            }
            let v_wit: Set[Y] satisfy {
                Y.is_open(v_wit) and v = v_wit.intersection(set_image(s, f))
            }
            let up: Set[X] = set_preimage(f, u_wit).intersection(s)
            let vp: Set[X] = set_preimage(f, v_wit).intersection(s)
            continuous_open_preimage[X, Y](f, u_wit)
            X.is_open(set_preimage(f, u_wit))
            subspace_open_from_ambient[X](s, set_preimage(f, u_wit))
            is_open_in_subspace[X](s, set_preimage(f, u_wit).intersection(s))
            is_open_in_subspace[X](s, up)
            continuous_open_preimage[X, Y](f, v_wit)
            X.is_open(set_preimage(f, v_wit))
            subspace_open_from_ambient[X](s, set_preimage(f, v_wit))
            is_open_in_subspace[X](s, set_preimage(f, v_wit).intersection(s))
            is_open_in_subspace[X](s, vp)
            u != Set[Y].empty_set
            nonempty_from_neq_empty_generic[Y](u)
            let y0: Y satisfy {
                u.contains(y0)
            }
            u = u_wit.intersection(set_image(s, f))
            u_wit.intersection(set_image(s, f)).contains(y0)
            intersection_contains_eq(u_wit, set_image(s, f), y0)
            set_image(s, f).contains(y0)
            set_image_contains_witness(s, f, y0)
            let x0: X satisfy {
                s.contains(x0) and y0 = f(x0)
            }
            u = u_wit.intersection(set_image(s, f))
            u_wit.intersection(set_image(s, f)).contains(y0)
            intersection_contains_eq(u_wit, set_image(s, f), y0)
            u_wit.contains(y0)
            set_preimage_contains_eq(f, u_wit, x0)
            set_preimage(f, u_wit).contains(x0)
            s.contains(x0)
            intersection_contains_intro(set_preimage(f, u_wit), s, x0)
            up.contains(x0)
            nonempty_of_contains(up, x0)
            up != Set[X].empty_set
            v != Set[Y].empty_set
            nonempty_from_neq_empty_generic[Y](v)
            let y1: Y satisfy {
                v.contains(y1)
            }
            v = v_wit.intersection(set_image(s, f))
            v_wit.intersection(set_image(s, f)).contains(y1)
            intersection_contains_eq(v_wit, set_image(s, f), y1)
            set_image(s, f).contains(y1)
            set_image_contains_witness(s, f, y1)
            let x1: X satisfy {
                s.contains(x1) and y1 = f(x1)
            }
            v = v_wit.intersection(set_image(s, f))
            v_wit.intersection(set_image(s, f)).contains(y1)
            intersection_contains_eq(v_wit, set_image(s, f), y1)
            v_wit.contains(y1)
            set_preimage_contains_eq(f, v_wit, x1)
            set_preimage(f, v_wit).contains(x1)
            s.contains(x1)
            intersection_contains_intro(set_preimage(f, v_wit), s, x1)
            vp.contains(x1)
            nonempty_of_contains(vp, x1)
            vp != Set[X].empty_set
            u.is_disjoint(v)
            up.is_disjoint(vp) = forall(w: X) {
                not (up.contains(w) and vp.contains(w))
            }
            forall(w: X) {
                if up.contains(w) and vp.contains(w) {
                    intersection_contains_eq(set_preimage(f, u_wit), s, w)
                    set_preimage(f, u_wit).contains(w)
                    set_preimage_contains_eq(f, u_wit, w)
                    u_wit.contains(f(w))
                    intersection_contains_eq(set_preimage(f, v_wit), s, w)
                    set_preimage(f, v_wit).contains(w)
                    set_preimage_contains_eq(f, v_wit, w)
                    v_wit.contains(f(w))
                    maps_into_set_image(s, f, w)
                    set_image(s, f).contains(f(w))
                    intersection_contains_intro(u_wit, set_image(s, f), f(w))
                    u_wit.intersection(set_image(s, f)).contains(f(w))
                    u.contains(f(w))
                    intersection_contains_intro(v_wit, set_image(s, f), f(w))
                    v_wit.intersection(set_image(s, f)).contains(f(w))
                    v.contains(f(w))
                    u.is_disjoint(v)
                    not (u.contains(f(w)) and v.contains(f(w)))
                    false
                }
                not (up.contains(w) and vp.contains(w))
            }
            up.is_disjoint(vp)
            u.union(v) = set_image(s, f)
            forall(w: X) {
                if s.contains(w) {
                    maps_into_set_image(s, f, w)
                    set_image(s, f).contains(f(w))
                    u.union(v).contains(f(w))
                    union_contains_eq(u, v, f(w))
                    if u.contains(f(w)) {
                        u = u_wit.intersection(set_image(s, f))
                        u_wit.intersection(set_image(s, f)).contains(f(w))
                        intersection_contains_eq(u_wit, set_image(s, f), f(w))
                        u_wit.contains(f(w))
                        set_preimage_contains_eq(f, u_wit, w)
                        set_preimage(f, u_wit).contains(w)
                        s.contains(w)
                        intersection_contains_intro(set_preimage(f, u_wit), s, w)
                        up.contains(w)
                        union_contains_eq(up, vp, w)
                        up.union(vp).contains(w)
                    } else {
                        v.contains(f(w))
                        v = v_wit.intersection(set_image(s, f))
                        v_wit.intersection(set_image(s, f)).contains(f(w))
                        intersection_contains_eq(v_wit, set_image(s, f), f(w))
                        v_wit.contains(f(w))
                        set_preimage_contains_eq(f, v_wit, w)
                        set_preimage(f, v_wit).contains(w)
                        s.contains(w)
                        intersection_contains_intro(set_preimage(f, v_wit), s, w)
                        vp.contains(w)
                        union_contains_eq(up, vp, w)
                        up.union(vp).contains(w)
                    }
                }
            }
            s.subset(up.union(vp))
            forall(w: X) {
                if up.union(vp).contains(w) {
                    union_contains_eq(up, vp, w)
                    if up.contains(w) {
                        intersection_contains_eq(set_preimage(f, u_wit), s, w)
                        s.contains(w)
                    } else {
                        intersection_contains_eq(set_preimage(f, v_wit), s, w)
                        s.contains(w)
                    }
                }
            }
            up.union(vp).subset(s)
            subset_antisymm(s, up.union(vp))
            up.union(vp) = s
            exists(uu: Set[X], vv: Set[X]) {
                is_open_in_subspace[X](s, uu) and is_open_in_subspace[X](s, vv) and uu != Set[X].empty_set and vv != Set[X].empty_set and uu.is_disjoint(vv) and uu.union(vv) = s
            }
            is_connected_set_iff[X](s)
            is_connected_set[X](s) = not exists(uu: Set[X], vv: Set[X]) {
                is_open_in_subspace[X](s, uu) and is_open_in_subspace[X](s, vv) and uu != Set[X].empty_set and vv != Set[X].empty_set and uu.is_disjoint(vv) and uu.union(vv) = s
            }
            not exists(uu: Set[X], vv: Set[X]) {
                is_open_in_subspace[X](s, uu) and is_open_in_subspace[X](s, vv) and uu != Set[X].empty_set and vv != Set[X].empty_set and uu.is_disjoint(vv) and uu.union(vv) = s
            }
            false
        }
        not exists(u: Set[Y], v: Set[Y]) {
            is_open_in_subspace[Y](set_image(s, f), u) and is_open_in_subspace[Y](set_image(s, f), v) and u != Set[Y].empty_set and v != Set[Y].empty_set and u.is_disjoint(v) and u.union(v) = set_image(s, f)
        }
        is_connected_set[Y](set_image(s, f))
    }
}
