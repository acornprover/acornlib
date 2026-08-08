from list import List
from data.basic.set import Set, set_image, set_preimage, subset_contains, subset_contains_eq,
    intersection_contains_eq, intersection_comm
from analysis.topology.topological_space import TopologicalSpace, big_union, big_union_contains_eq,
    big_union_contains_of_member, is_closed, is_open_cover, is_open_cover_iff,
    is_compact, is_subfamily, list_union_of_sets, is_closed_in_subspace,
    subspace_closed_from_ambient, subspace_closed_subset_carrier, is_continuous
from analysis.topology.compact_union import compact_open_cover_has_finite_subcover
from analysis.topology.compact_closed_subset import cover_adjoin_complement,
    cover_adjoin_complement_open_sets, cover_restrict_to_family,
    cover_restrict_to_family_is_subfamily, cover_restrict_to_family_covers_closed_member,
    closed_subset_of_compact_is_compact
from analysis.topology.compact_image import continuous_image_compact
from analysis.topology.continuity_closed import continuous_iff_preimage_closed

/// A finite subcover of a subset by an augmented family restricts to a finite
/// subcover from the original family, provided the subset lies in the closed member.
theorem cover_restrict_to_family_finite_subcover_of_subset[T](
    s: Set[T],
    closed: Set[T],
    c: Set[T] -> Bool,
    items: List[Set[T]]
) {
    s.subset(closed) and is_subfamily[T](cover_adjoin_complement[T](closed, c), items)
        and s.subset(list_union_of_sets[T](items))
        implies exists(filtered: List[Set[T]]) {
            is_subfamily[T](c, filtered) and s.subset(list_union_of_sets[T](filtered))
        }
} by {
    if s.subset(closed) and is_subfamily[T](cover_adjoin_complement[T](closed, c), items)
        and s.subset(list_union_of_sets[T](items)) {
        let filtered: List[Set[T]] = cover_restrict_to_family[T](c, items)
        cover_restrict_to_family_is_subfamily[T](c, items)
        forall(x: T) {
            if s.contains(x) {
                subset_contains[T](s, closed, x)
                subset_contains[T](s, list_union_of_sets[T](items), x)
                list_union_of_sets[T](items).contains(x)
                cover_restrict_to_family_covers_closed_member[T](closed, c, items, x)
                list_union_of_sets[T](filtered).contains(x)
            }
        }
        subset_contains_eq[T](s, list_union_of_sets[T](filtered))
        s.subset(list_union_of_sets[T](filtered))
        exists(result: List[Set[T]]) {
            is_subfamily[T](c, result) and s.subset(list_union_of_sets[T](result))
        }
    }
}

/// If `s` is the intersection of a carrier with a closed ambient set, then an
/// open cover of `s` extends to an open cover of the carrier by adjoining the
/// ambient closed set's complement.
theorem cover_adjoin_complement_is_open_cover_of_subspace_closed_subset_ambient[T: TopologicalSpace](
    carrier: Set[T],
    s: Set[T],
    closed: Set[T],
    c: Set[T] -> Bool
) {
    is_closed[T](closed) and is_open_cover[T](c, s) and s = closed.intersection(carrier)
        implies is_open_cover[T](cover_adjoin_complement[T](closed, c), carrier)
} by {
    if is_closed[T](closed) and is_open_cover[T](c, s) and s = closed.intersection(carrier) {
        let d: Set[T] -> Bool = cover_adjoin_complement[T](closed, c)
        is_open_cover_iff[T](c, s)
        is_open_cover[T](c, s) = ((forall(u: Set[T]) { c(u) implies T.is_open(u) })
            and s.subset(big_union[T](c)))
        forall(u: Set[T]) { c(u) implies T.is_open(u) }
        s.subset(big_union[T](c))
        cover_adjoin_complement_open_sets[T](closed, c)
        forall(u: Set[T]) { d(u) implies T.is_open(u) }
        forall(x: T) {
            if carrier.contains(x) {
                if closed.contains(x) {
                    intersection_contains_eq[T](closed, carrier, x)
                    subset_contains[T](s, big_union[T](c), x)
                    big_union_contains_eq[T](c, x)
                    let u: Set[T] satisfy {
                        c(u) and u.contains(x)
                    }
                    cover_adjoin_complement[T](closed, c, u) = (c(u) or u = closed.c)
                    big_union_contains_of_member[T](d, u, x)
                    big_union[T](d).contains(x)
                } else {
                    d(closed.c)
                    big_union_contains_of_member[T](d, closed.c, x)
                    big_union[T](d).contains(x)
                }
            }
        }
        subset_contains_eq[T](carrier, big_union[T](d))
        carrier.subset(big_union[T](d))
        is_open_cover_iff[T](d, carrier)
        is_open_cover[T](d, carrier)
        is_open_cover[T](cover_adjoin_complement[T](closed, c), carrier)
    }
}

/// A subset that is closed in a compact ambient subspace is compact.
theorem compact_of_subspace_closed_subset[T: TopologicalSpace](carrier: Set[T], s: Set[T]) {
    is_compact[T](carrier) and is_closed_in_subspace[T](carrier, s) implies is_compact[T](s)
} by {
    if is_compact[T](carrier) and is_closed_in_subspace[T](carrier, s) {
        let closed: Set[T] satisfy {
            is_closed[T](closed) and s = closed.intersection(carrier)
        }
        forall(c: Set[T] -> Bool) {
            if is_open_cover[T](c, s) {
                let d: Set[T] -> Bool = cover_adjoin_complement[T](closed, c)
                cover_adjoin_complement_is_open_cover_of_subspace_closed_subset_ambient[T](
                    carrier, s, closed, c)
                compact_open_cover_has_finite_subcover[T](d, carrier)
                let items: List[Set[T]] satisfy {
                    is_subfamily[T](d, items) and carrier.subset(list_union_of_sets[T](items))
                }
                subspace_closed_subset_carrier[T](carrier, s)
                forall(x: T) {
                    if s.contains(x) {
                        subset_contains[T](s, carrier, x)
                        subset_contains[T](carrier, list_union_of_sets[T](items), x)
                        list_union_of_sets[T](items).contains(x)
                    }
                }
                subset_contains_eq[T](s, list_union_of_sets[T](items))
                s.subset(list_union_of_sets[T](items))
                forall(x: T) {
                    if s.contains(x) {
                        intersection_contains_eq[T](closed, carrier, x)
                        closed.contains(x)
                    }
                }
                subset_contains_eq[T](s, closed)
                s.subset(closed)
                cover_restrict_to_family_finite_subcover_of_subset[T](s, closed, c, items)
                exists(filtered: List[Set[T]]) {
                    is_subfamily[T](c, filtered) and s.subset(list_union_of_sets[T](filtered))
                }
            }
        }
    }
}

/// The intersection of a compact set with a closed set is compact.
theorem compact_inter_closed_is_compact[T: TopologicalSpace](carrier: Set[T], closed: Set[T]) {
    is_compact[T](carrier) and is_closed[T](closed) implies is_compact[T](carrier.intersection(closed))
} by {
    if is_compact[T](carrier) and is_closed[T](closed) {
        subspace_closed_from_ambient[T](carrier, closed)
        compact_of_subspace_closed_subset[T](carrier, closed.intersection(carrier))
        intersection_comm[T](carrier, closed)
        is_compact[T](carrier.intersection(closed))
    }
}

/// A closed subset of a compact domain has compact image under a continuous map.
theorem continuous_image_closed_subset_of_compact_is_compact[X: TopologicalSpace, Y: TopologicalSpace](
    f: X -> Y,
    s: Set[X],
    carrier: Set[X]
) {
    is_continuous[X, Y](f) and is_closed[X](s) and s.subset(carrier) and is_compact[X](carrier)
        implies is_compact[Y](set_image[X, Y](s, f))
} by {
    if is_continuous[X, Y](f) and is_closed[X](s) and s.subset(carrier) and is_compact[X](carrier) {
        closed_subset_of_compact_is_compact[X](s, carrier)
        continuous_image_compact[X, Y](f, s)
        is_compact[Y](set_image[X, Y](s, f))
    }
}

/// The image of a subspace-closed subset of a compact domain is compact under a continuous map.
theorem continuous_image_subspace_closed_subset_compact[X: TopologicalSpace, Y: TopologicalSpace](
    f: X -> Y,
    carrier: Set[X],
    s: Set[X]
) {
    is_continuous[X, Y](f) and is_compact[X](carrier) and is_closed_in_subspace[X](carrier, s)
        implies is_compact[Y](set_image[X, Y](s, f))
} by {
    if is_continuous[X, Y](f) and is_compact[X](carrier) and is_closed_in_subspace[X](carrier, s) {
        compact_of_subspace_closed_subset[X](carrier, s)
        continuous_image_compact[X, Y](f, s)
        is_compact[Y](set_image[X, Y](s, f))
    }
}

/// The preimage of a closed set under a continuous map is compact when restricted to a compact domain.
theorem compact_inter_preimage_closed_is_compact[X: TopologicalSpace, Y: TopologicalSpace](
    f: X -> Y,
    carrier: Set[X],
    closed: Set[Y]
) {
    is_continuous[X, Y](f) and is_compact[X](carrier) and is_closed[Y](closed)
        implies is_compact[X](carrier.intersection(set_preimage[X, Y](f, closed)))
} by {
    if is_continuous[X, Y](f) and is_compact[X](carrier) and is_closed[Y](closed) {
        continuous_iff_preimage_closed[X, Y](f)
        if not is_closed[X](set_preimage[X, Y](f, closed)) {
            false
        }
        compact_inter_closed_is_compact[X](carrier, set_preimage[X, Y](f, closed))
        is_compact[X](carrier.intersection(set_preimage[X, Y](f, closed)))
    }
}
