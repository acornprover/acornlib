from data.basic.set import Set, subset_antisymm, union_with_empty_is_self,
    difference_of_self_is_empty
from analysis.topology.topological_space import TopologicalSpace, is_closed, closure, interior,
    frontier, closure_closed, closed_eq_closure, subset_closure,
    interior_subset, open_iff_eq_interior, is_clopen, is_clopen_intro,
    is_clopen_imp_open, is_connected, is_connected_via_clopen,
    is_connected_via_clopen_iff
from analysis.topology.frontier_props import closure_eq_interior_union_frontier
from analysis.topology.frontier_open_closed import open_frontier_eq_closure_diff_self
from analysis.topology.connectedness import connected_imp_via_clopen,
    via_clopen_nonempty_clopen_universal

/// An open and closed set is clopen.
theorem is_clopen_of_open_closed[T: TopologicalSpace](s: Set[T]) {
    T.is_open(s) and is_closed[T](s) implies is_clopen[T](T.is_open, s)
} by {
    if T.is_open(s) and is_closed[T](s) {
        T.is_open(s) and T.is_open(s.c)
        is_clopen_intro[T](T.is_open, s)
    }
}

/// For a topological space, clopen is equivalent to open and closed.
theorem is_clopen_iff_open_closed[T: TopologicalSpace](s: Set[T]) {
    is_clopen[T](T.is_open, s) = (T.is_open(s) and is_closed[T](s))
} by {
    if is_clopen[T](T.is_open, s) {
        is_clopen_imp_open[T](T.is_open, s)
        is_closed[T](s) = T.is_open(s.c)
        is_closed[T](s)
        T.is_open(s) and is_closed[T](s)
    }
    if T.is_open(s) and is_closed[T](s) {
        is_clopen_of_open_closed[T](s)
        is_clopen[T](T.is_open, s)
    }
}

/// A clopen set has empty frontier.
theorem frontier_empty_of_clopen[T: TopologicalSpace](s: Set[T]) {
    is_clopen[T](T.is_open, s) implies frontier[T](s) = Set[T].empty_set
} by {
    if is_clopen[T](T.is_open, s) {
        is_clopen_iff_open_closed[T](s)
        T.is_open(s) and is_closed[T](s)
        open_frontier_eq_closure_diff_self[T](s)
        frontier[T](s) = closure[T](s).difference(s)
        closed_eq_closure[T](s)
        s = closure[T](s)
        closure[T](s) = s
        frontier[T](s) = s.difference(s)
        difference_of_self_is_empty[T](s)
        s.difference(s) = Set[T].empty_set
        frontier[T](s) = Set[T].empty_set
    }
}

/// If a set has empty frontier, then it is clopen.
theorem clopen_of_frontier_empty[T: TopologicalSpace](s: Set[T]) {
    frontier[T](s) = Set[T].empty_set implies is_clopen[T](T.is_open, s)
} by {
    if frontier[T](s) = Set[T].empty_set {
        closure_eq_interior_union_frontier[T](s)
        closure[T](s) = interior[T](s).union(frontier[T](s))
        closure[T](s) = interior[T](s).union(Set[T].empty_set)
        union_with_empty_is_self[T](interior[T](s))
        interior[T](s).union(Set[T].empty_set) = interior[T](s)
        closure[T](s) = interior[T](s)
        subset_closure[T](s)
        s.subset(closure[T](s))
        s.subset(interior[T](s))
        interior_subset[T](s)
        interior[T](s).subset(s)
        subset_antisymm[T](s, interior[T](s))
        s = interior[T](s)
        open_iff_eq_interior[T](s)
        T.is_open(s)
        interior[T](s) = s
        closure[T](s) = s
        closure_closed[T](s)
        is_closed[T](closure[T](s))
        is_closed[T](s)
        is_clopen_of_open_closed[T](s)
        is_clopen[T](T.is_open, s)
    }
}

/// A set has empty frontier exactly when it is clopen.
theorem frontier_empty_iff_clopen[T: TopologicalSpace](s: Set[T]) {
    (frontier[T](s) = Set[T].empty_set) = is_clopen[T](T.is_open, s)
} by {
    if frontier[T](s) = Set[T].empty_set {
        clopen_of_frontier_empty[T](s)
        is_clopen[T](T.is_open, s)
    }
    if is_clopen[T](T.is_open, s) {
        frontier_empty_of_clopen[T](s)
        frontier[T](s) = Set[T].empty_set
    }
}

/// In a connected space, a set with empty frontier is trivial.
theorem connected_frontier_empty_trivial[T: TopologicalSpace](s: Set[T]) {
    is_connected[T](T.is_open) and frontier[T](s) = Set[T].empty_set
        implies s = Set[T].empty_set or s = Set[T].universal_set
} by {
    if is_connected[T](T.is_open) and frontier[T](s) = Set[T].empty_set {
        connected_imp_via_clopen[T](T.is_open)
        is_connected_via_clopen[T](T.is_open)
        clopen_of_frontier_empty[T](s)
        is_clopen[T](T.is_open, s)
        is_connected_via_clopen_iff[T](T.is_open)
        if not (s = Set[T].empty_set or s = Set[T].universal_set) {
            false
        }
        s = Set[T].empty_set or s = Set[T].universal_set
    }
}

/// In a connected space, a nonempty set with empty frontier is universal.
theorem connected_nonempty_frontier_empty_universal[T: TopologicalSpace](s: Set[T]) {
    is_connected[T](T.is_open) and frontier[T](s) = Set[T].empty_set and s != Set[T].empty_set
        implies s = Set[T].universal_set
} by {
    if is_connected[T](T.is_open) and frontier[T](s) = Set[T].empty_set and s != Set[T].empty_set {
        connected_imp_via_clopen[T](T.is_open)
        is_connected_via_clopen[T](T.is_open)
        clopen_of_frontier_empty[T](s)
        is_clopen[T](T.is_open, s)
        via_clopen_nonempty_clopen_universal[T](T.is_open, s)
        s = Set[T].universal_set
    }
}
