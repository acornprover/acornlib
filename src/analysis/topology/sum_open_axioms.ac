/// Sum topology and continuity of the coproduct injections.

from data.basic.set import Set, set_ext, set_preimage, set_preimage_contains_eq, preimage_contains
from sum_type import Sum, sum_inl, sum_inr
from analysis.topology.topological_space import TopologicalSpace, is_continuous, is_open_in_sum,
    sum_left_preimage, sum_right_preimage, sum_left_preimage_contains_eq,
    sum_right_preimage_contains_eq, is_open_in_sum_left, is_open_in_sum_right

/// The sum topology on `Sum[X, Y]`: a set is open when both injection preimages are open.
instance Sum[X: TopologicalSpace, Y: TopologicalSpace]: TopologicalSpace {
    let is_open: Set[Sum[X, Y]] -> Bool = is_open_in_sum[X, Y]
}

/// Openness in the sum topology coincides with the predicate `is_open_in_sum`.
theorem sum_is_open_eq[X: TopologicalSpace, Y: TopologicalSpace](w: Set[Sum[X, Y]]) {
    Sum[X, Y].is_open(w) = is_open_in_sum[X, Y](w)
}

/// The preimage of a sum set under the left injection is its left preimage.
theorem sum_inl_preimage_eq[X, Y](w: Set[Sum[X, Y]]) {
    set_preimage(sum_inl[X, Y], w) = sum_left_preimage(w)
} by {
    forall(x: X) {
        set_preimage_contains_eq(sum_inl[X, Y], w, x)
        sum_inl[X, Y](x) = Sum.inl[X, Y](x)
        preimage_contains(sum_inl[X, Y], w, x) = w.contains(Sum.inl[X, Y](x))
        sum_left_preimage_contains_eq(w, x)
        set_preimage(sum_inl[X, Y], w).contains(x) = sum_left_preimage(w).contains(x)
    }
    set_ext(set_preimage(sum_inl[X, Y], w), sum_left_preimage(w))
}

/// The preimage of a sum set under the right injection is its right preimage.
theorem sum_inr_preimage_eq[X, Y](w: Set[Sum[X, Y]]) {
    set_preimage(sum_inr[X, Y], w) = sum_right_preimage(w)
} by {
    forall(y: Y) {
        set_preimage_contains_eq(sum_inr[X, Y], w, y)
        sum_inr[X, Y](y) = Sum.inr[X, Y](y)
        preimage_contains(sum_inr[X, Y], w, y) = w.contains(Sum.inr[X, Y](y))
        sum_right_preimage_contains_eq(w, y)
        set_preimage(sum_inr[X, Y], w).contains(y) = sum_right_preimage(w).contains(y)
    }
    set_ext(set_preimage(sum_inr[X, Y], w), sum_right_preimage(w))
}

/// The left injection into a topological sum is continuous.
theorem sum_inl_continuous[X: TopologicalSpace, Y: TopologicalSpace] {
    is_continuous(sum_inl[X, Y])
} by {
    forall(w: Set[Sum[X, Y]]) {
        if Sum[X, Y].is_open(w) {
            sum_is_open_eq[X, Y](w)
            is_open_in_sum[X, Y](w)
            is_open_in_sum_left[X, Y](w)
            X.is_open(sum_left_preimage(w))
            sum_inl_preimage_eq[X, Y](w)
            X.is_open(set_preimage(sum_inl[X, Y], w))
        }
    }
    is_continuous(sum_inl[X, Y]) = forall(w: Set[Sum[X, Y]]) {
        Sum[X, Y].is_open(w) implies X.is_open(set_preimage(sum_inl[X, Y], w))
    }
}

/// The right injection into a topological sum is continuous.
theorem sum_inr_continuous[X: TopologicalSpace, Y: TopologicalSpace] {
    is_continuous(sum_inr[X, Y])
} by {
    forall(w: Set[Sum[X, Y]]) {
        if Sum[X, Y].is_open(w) {
            sum_is_open_eq[X, Y](w)
            is_open_in_sum[X, Y](w)
            is_open_in_sum_right[X, Y](w)
            Y.is_open(sum_right_preimage(w))
            sum_inr_preimage_eq[X, Y](w)
            Y.is_open(set_preimage(sum_inr[X, Y], w))
        }
    }
    is_continuous(sum_inr[X, Y]) = forall(w: Set[Sum[X, Y]]) {
        Sum[X, Y].is_open(w) implies Y.is_open(set_preimage(sum_inr[X, Y], w))
    }
}
