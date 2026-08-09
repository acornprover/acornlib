/// Borel measurable functions on the real line and their closure properties.
///
/// This file instantiates the Borel sigma-algebra on `Real` (via the standard
/// topology, in which a set is open exactly when every point of the set is an
/// interior point) and proves that Borel measurability is closed under
/// composition, that constant functions and the identity are Borel measurable,
/// and that every continuous function is Borel measurable.
from data.basic.functions import compose, identity_fn
from data.basic.set import Set, set_preimage, set_preimage_contains_eq,
    empty_set_contains_eq, universal_set_contains_eq, intersection_contains_eq,
    intersection_contains_intro
from real import Real, continuous, continuous_at, continuous_condition
from order import lt_of_lt_of_lte
from analysis.topology.topological_space import TopologicalSpace, big_union,
    big_union_contains_eq, big_union_contains_of_member
from analysis.topology.measurable_space import is_measurable_fn,
    is_measurable_fn_into_generated, generated_sigma_algebra
from analysis.topology.borel import borel_sigma_algebra,
    borel_sigma_algebra_eq_generated_open, borel_sigma_algebra_open_is_measurable,
    borel_sigma_algebra_compose_is_measurable,
    borel_sigma_algebra_const_is_measurable, borel_sigma_algebra_identity_is_measurable

/// True if `x` is an interior point of the set `s` in the standard topology
/// on the real line: `x` lies in `s` and some positive ball around `x` is
/// contained in `s`.
define real_is_interior_point(s: Set[Real], x: Real) -> Bool {
    s.contains(x) and exists(eps: Real) {
        eps.is_positive and forall(y: Real) {
            y.is_close(x, eps) implies s.contains(y)
        }
    }
}

/// True if `s` is open in the standard topology on the real line: every point
/// of `s` is an interior point.
define real_is_open(s: Set[Real]) -> Bool {
    forall(x: Real) {
        s.contains(x) implies real_is_interior_point(s, x)
    }
}

/// A positive ball contained in a set makes its center an interior point.
theorem real_is_interior_point_intro(s: Set[Real], x: Real, eps: Real) {
    s.contains(x) and eps.is_positive and forall(y: Real) {
        y.is_close(x, eps) implies s.contains(y)
    } implies real_is_interior_point(s, x)
} by {
    if s.contains(x) and eps.is_positive and forall(y: Real) {
        y.is_close(x, eps) implies s.contains(y)
    } {
        exists(e: Real) {
            e.is_positive and forall(y: Real) {
                y.is_close(x, e) implies s.contains(y)
            }
        }
        real_is_interior_point(s, x) = (s.contains(x) and exists(e: Real) {
            e.is_positive and forall(y: Real) {
                y.is_close(x, e) implies s.contains(y)
            }
        })
        real_is_interior_point(s, x)
    }
}

/// An open set makes each of its points an interior point.
theorem real_open_is_interior_point(s: Set[Real], x: Real) {
    real_is_open(s) and s.contains(x) implies real_is_interior_point(s, x)
} by {
    if real_is_open(s) and s.contains(x) {
        real_is_open(s) = forall(y: Real) {
            s.contains(y) implies real_is_interior_point(s, y)
        }
        real_is_interior_point(s, x)
    }
}

/// The empty set is open in the standard topology on the real line.
theorem real_open_empty {
    real_is_open(Set[Real].empty_set)
} by {
    forall(x: Real) {
        if Set[Real].empty_set.contains(x) {
            empty_set_contains_eq[Real](x)
            false
        }
    }
    real_is_open(Set[Real].empty_set) = forall(x: Real) {
        Set[Real].empty_set.contains(x) implies real_is_interior_point(Set[Real].empty_set, x)
    }
    real_is_open(Set[Real].empty_set)
}

/// The real line is open in the standard topology.
theorem real_open_universal {
    real_is_open(Set[Real].universal_set)
} by {
    forall(x: Real) {
        if Set[Real].universal_set.contains(x) {
            Real.1.is_positive
            forall(y: Real) {
                if y.is_close(x, Real.1) {
                    universal_set_contains_eq[Real](y)
                    Set[Real].universal_set.contains(y)
                }
            }
            real_is_interior_point_intro(Set[Real].universal_set, x, Real.1)
            real_is_interior_point(Set[Real].universal_set, x)
        }
    }
    real_is_open(Set[Real].universal_set) = forall(x: Real) {
        Set[Real].universal_set.contains(x) implies real_is_interior_point(Set[Real].universal_set, x)
    }
    real_is_open(Set[Real].universal_set)
}

/// A ball of radius `eps1` contained in `s` also contains every ball of radius
/// no larger than `eps1`.
theorem real_smaller_ball_subset_left(s: Set[Real], x: Real, eps: Real, eps1: Real) {
    eps <= eps1 and forall(y: Real) {
        y.is_close(x, eps1) implies s.contains(y)
    } implies forall(y: Real) {
        y.is_close(x, eps) implies s.contains(y)
    }
} by {
    if eps <= eps1 and forall(y: Real) { y.is_close(x, eps1) implies s.contains(y) } {
        forall(y: Real) {
            if y.is_close(x, eps) {
                (y - x).abs < eps
                lt_of_lt_of_lte((y - x).abs, eps, eps1)
                (y - x).abs < eps1
                y.is_close(x, eps1)
                s.contains(y)
            }
        }
    }
}

/// Interior points are closed under binary intersection.
theorem real_is_interior_point_intersection(s: Set[Real], t: Set[Real], x: Real) {
    real_is_interior_point(s, x) and real_is_interior_point(t, x) implies
    real_is_interior_point(s.intersection(t), x)
} by {
    if real_is_interior_point(s, x) and real_is_interior_point(t, x) {
        s.contains(x)
        t.contains(x)
        intersection_contains_intro(s, t, x)
        real_is_interior_point(s, x) = (s.contains(x) and exists(eps: Real) {
            eps.is_positive and forall(y: Real) {
                y.is_close(x, eps) implies s.contains(y)
            }
        })
        real_is_interior_point(t, x) = (t.contains(x) and exists(eps: Real) {
            eps.is_positive and forall(y: Real) {
                y.is_close(x, eps) implies t.contains(y)
            }
        })
        let eps1: Real satisfy {
            eps1.is_positive and forall(y: Real) {
                y.is_close(x, eps1) implies s.contains(y)
            }
        }
        let eps2: Real satisfy {
            eps2.is_positive and forall(y: Real) {
                y.is_close(x, eps2) implies t.contains(y)
            }
        }
        let eps = eps1.min(eps2)
        eps.is_positive
        eps <= eps1
        eps <= eps2
        real_smaller_ball_subset_left(s, x, eps, eps1)
        forall(y: Real) {
            y.is_close(x, eps) implies s.contains(y)
        }
        real_smaller_ball_subset_left(t, x, eps, eps2)
        forall(y: Real) {
            y.is_close(x, eps) implies t.contains(y)
        }
        forall(y: Real) {
            if y.is_close(x, eps) {
                s.contains(y)
                t.contains(y)
                intersection_contains_intro(s, t, y)
                s.intersection(t).contains(y)
            }
        }
        s.intersection(t).contains(x)
        eps.is_positive
        real_is_interior_point_intro(s.intersection(t), x, eps)
        real_is_interior_point(s.intersection(t), x)
    }
}

/// The intersection of two open sets is open in the standard topology.
theorem real_open_inter(s: Set[Real], t: Set[Real]) {
    real_is_open(s) and real_is_open(t) implies real_is_open(s.intersection(t))
} by {
    if real_is_open(s) and real_is_open(t) {
        forall(x: Real) {
            if s.intersection(t).contains(x) {
                intersection_contains_eq(s, t, x)
                s.contains(x)
                t.contains(x)
                real_open_is_interior_point(s, x)
                real_open_is_interior_point(t, x)
                real_is_interior_point_intersection(s, t, x)
                real_is_interior_point(s.intersection(t), x)
            }
        }
        real_is_open(s.intersection(t)) = forall(x: Real) {
            s.intersection(t).contains(x) implies real_is_interior_point(s.intersection(t), x)
        }
        real_is_open(s.intersection(t))
    }
}

/// An arbitrary union of open sets is open in the standard topology.
theorem real_open_big_union(c: Set[Real] -> Bool) {
    (forall(s: Set[Real]) { c(s) implies real_is_open(s) })
        implies real_is_open(big_union(c))
} by {
    if forall(s: Set[Real]) { c(s) implies real_is_open(s) } {
        forall(x: Real) {
            if big_union(c).contains(x) {
                big_union_contains_eq(c, x)
                exists(s: Set[Real]) {
                    c(s) and s.contains(x)
                }
                let s: Set[Real] satisfy {
                    c(s) and s.contains(x)
                }
                c(s) implies real_is_open(s)
                real_is_open(s)
                real_open_is_interior_point(s, x)
                real_is_interior_point(s, x)
                real_is_interior_point(s, x) = (s.contains(x) and exists(eps: Real) {
                    eps.is_positive and forall(y: Real) {
                        y.is_close(x, eps) implies s.contains(y)
                    }
                })
                exists(eps: Real) {
                    eps.is_positive and forall(y: Real) {
                        y.is_close(x, eps) implies s.contains(y)
                    }
                }
                let eps: Real satisfy {
                    eps.is_positive and forall(y: Real) {
                        y.is_close(x, eps) implies s.contains(y)
                    }
                }
                big_union(c).contains(x)
                eps.is_positive
                forall(y: Real) {
                    if y.is_close(x, eps) {
                        s.contains(y)
                        big_union_contains_of_member(c, s, y)
                        big_union(c).contains(y)
                    }
                }
                real_is_interior_point_intro(big_union(c), x, eps)
                real_is_interior_point(big_union(c), x)
            }
        }
        real_is_open(big_union(c)) = forall(x: Real) {
            big_union(c).contains(x) implies real_is_interior_point(big_union(c), x)
        }
        real_is_open(big_union(c))
    }
}

/// The standard topology on the real line: a set is open exactly when every
/// point of the set is an interior point.
instance Real: TopologicalSpace {
    let is_open: Set[Real] -> Bool = real_is_open
}

/// True if the function `f` on the real line is Borel measurable: the preimage
/// of every Borel set under `f` is Borel.
define is_borel_measurable_fn(f: Real -> Real) -> Bool {
    is_measurable_fn(f, borel_sigma_algebra[Real], borel_sigma_algebra[Real])
}

/// The composition of two Borel-measurable functions on the real line is
/// Borel-measurable.
theorem borel_measurable_fn_compose(f: Real -> Real, g: Real -> Real) {
    is_borel_measurable_fn(g) and is_borel_measurable_fn(f)
        implies is_borel_measurable_fn(compose(f, g))
} by {
    if is_borel_measurable_fn(g) and is_borel_measurable_fn(f) {
        is_borel_measurable_fn(g) =
            is_measurable_fn(g, borel_sigma_algebra[Real], borel_sigma_algebra[Real])
        is_borel_measurable_fn(f) =
            is_measurable_fn(f, borel_sigma_algebra[Real], borel_sigma_algebra[Real])
        is_measurable_fn(g, borel_sigma_algebra[Real], borel_sigma_algebra[Real]) and
            is_measurable_fn(f, borel_sigma_algebra[Real], borel_sigma_algebra[Real])
        borel_sigma_algebra_compose_is_measurable(f, g)
        is_measurable_fn(compose(f, g), borel_sigma_algebra[Real], borel_sigma_algebra[Real])
        is_borel_measurable_fn(compose(f, g)) =
            is_measurable_fn(compose(f, g), borel_sigma_algebra[Real], borel_sigma_algebra[Real])
        is_borel_measurable_fn(compose(f, g))
    }
}

/// Every constant function on the real line is Borel-measurable.
theorem borel_measurable_fn_const(c: Real) {
    is_borel_measurable_fn(function(x: Real) { c })
} by {
    borel_sigma_algebra_const_is_measurable[Real, Real](c)
    is_measurable_fn(function(x: Real) { c },
        borel_sigma_algebra[Real], borel_sigma_algebra[Real])
    is_borel_measurable_fn(function(x: Real) { c }) =
        is_measurable_fn(function(x: Real) { c },
            borel_sigma_algebra[Real], borel_sigma_algebra[Real])
    is_borel_measurable_fn(function(x: Real) { c })
}

/// The identity function on the real line is Borel-measurable.
theorem borel_measurable_fn_identity {
    is_borel_measurable_fn(identity_fn[Real])
} by {
    borel_sigma_algebra_identity_is_measurable[Real]
    is_measurable_fn(identity_fn[Real], borel_sigma_algebra[Real], borel_sigma_algebra[Real])
    is_borel_measurable_fn(identity_fn[Real]) =
        is_measurable_fn(identity_fn[Real], borel_sigma_algebra[Real], borel_sigma_algebra[Real])
    is_borel_measurable_fn(identity_fn[Real])
}

/// A continuous function on the real line pulls back open sets to open sets.
theorem continuous_real_open_preimage(f: Real -> Real) {
    continuous(f) implies forall(v: Set[Real]) {
        real_is_open(v) implies real_is_open(set_preimage(f, v))
    }
} by {
    if continuous(f) {
        continuous(f) = forall(x: Real) { continuous_at(f, x) }
        forall(v: Set[Real]) {
            if real_is_open(v) {
                real_is_open(v) = forall(z: Real) {
                    v.contains(z) implies real_is_interior_point(v, z)
                }
                forall(x: Real) {
                    if set_preimage(f, v).contains(x) {
                        set_preimage_contains_eq(f, v, x)
                        v.contains(f(x))
                        real_is_interior_point(v, f(x))
                        real_is_interior_point(v, f(x)) =
                            (v.contains(f(x)) and exists(eps: Real) {
                                eps.is_positive and forall(y: Real) {
                                    y.is_close(f(x), eps) implies v.contains(y)
                                }
                            })
                        exists(eps: Real) {
                            eps.is_positive and forall(y: Real) {
                                y.is_close(f(x), eps) implies v.contains(y)
                            }
                        }
                        let eps0: Real satisfy {
                            eps0.is_positive and forall(y: Real) {
                                y.is_close(f(x), eps0) implies v.contains(y)
                            }
                        }
                        continuous_at(f, x)
                        continuous_at(f, x) = forall(eps: Real) {
                            eps.is_positive implies exists(delta: Real) {
                                delta.is_positive and continuous_condition(f, x, delta, eps)
                            }
                        }
                        eps0.is_positive
                        exists(delta: Real) {
                            delta.is_positive and continuous_condition(f, x, delta, eps0)
                        }
                        let delta: Real satisfy {
                            delta.is_positive and continuous_condition(f, x, delta, eps0)
                        }
                        set_preimage(f, v).contains(x)
                        delta.is_positive
                        forall(y: Real) {
                            if y.is_close(x, delta) {
                                continuous_condition(f, x, delta, eps0) =
                                    forall(x1: Real) {
                                        x1.is_close(x, delta) implies
                                            f(x1).is_close(f(x), eps0)
                                    }
                                f(y).is_close(f(x), eps0)
                                v.contains(f(y))
                                set_preimage_contains_eq(f, v, y)
                                set_preimage(f, v).contains(y)
                            }
                        }
                        real_is_interior_point_intro(set_preimage(f, v), x, delta)
                        real_is_interior_point(set_preimage(f, v), x)
                    }
                }
                real_is_open(set_preimage(f, v)) = forall(x: Real) {
                    set_preimage(f, v).contains(x) implies
                        real_is_interior_point(set_preimage(f, v), x)
                }
                real_is_open(set_preimage(f, v))
            }
        }
    }
}


/// A continuous function on the real line is Borel measurable.
theorem continuous_imp_borel_measurable(f: Real -> Real) {
    continuous(f) implies is_borel_measurable_fn(f)
} by {
    if continuous(f) {
        continuous_real_open_preimage(f)
        forall(t: Set[Real]) {
            if Real.is_open(t) {
                real_is_open(set_preimage(f, t))
                borel_sigma_algebra_open_is_measurable[Real](set_preimage(f, t))
                borel_sigma_algebra[Real].is_measurable(set_preimage(f, t))
            }
        }
        is_measurable_fn_into_generated(f, borel_sigma_algebra[Real],
            function(s: Set[Real]) { Real.is_open(s) })
        is_measurable_fn(f, borel_sigma_algebra[Real],
            generated_sigma_algebra(function(s: Set[Real]) { Real.is_open(s) }))
        borel_sigma_algebra_eq_generated_open[Real]
        borel_sigma_algebra[Real] =
            generated_sigma_algebra(function(s: Set[Real]) { Real.is_open(s) })
        is_measurable_fn(f, borel_sigma_algebra[Real], borel_sigma_algebra[Real])
        is_borel_measurable_fn(f) =
            is_measurable_fn(f, borel_sigma_algebra[Real], borel_sigma_algebra[Real])
        is_borel_measurable_fn(f)
    }
}
