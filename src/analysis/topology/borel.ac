from data.basic.functions import compose, identity_fn
from nat import Nat
from data.basic.set import Set, set_preimage, set_preimage_compose,
    compl_of_compl_is_self, indexed_union, indexed_intersection
from analysis.topology.measurable_space import SigmaAlgebra,
    generated_sigma_algebra, generated_sigma_algebra_is_measurable_eq,
    generated_sigma_algebra_contains_generator, generated_measurable,
    is_measurable_fn, is_measurable_fn_into_generated,
    is_measurable_fn_preimage_measurable,
    is_measurable_fn_const, is_measurable_fn_identity, is_measurable_fn_compose,
    sigma_algebra_measurable_compl, sigma_algebra_le,
    sigma_algebra_le_antisymm, generated_sigma_algebra_is_smallest,
    sigma_algebra_measurable_indexed_union,
    sigma_algebra_measurable_indexed_intersection,
    indicator_fn, indicator_fn_is_measurable_iff,
    set_measurable_of_indicator_fn_measurable, indicator_fn_is_measurable_fn,
    preimage_sigma_algebra, preimage_generator,
    preimage_sigma_algebra_generated_eq,
    is_measurable_in_subspace, trace_sigma_algebra,
    trace_sigma_algebra_measurable_of_subspace,
    subspace_indicator_fn_is_measurable
from analysis.topology.topological_space import TopologicalSpace, is_closed, is_continuous,
    is_open_in_subspace, is_closed_in_subspace

/// The Borel sigma-algebra on a topological space: the smallest
/// sigma-algebra containing every open set.
let borel_sigma_algebra[T: TopologicalSpace]: SigmaAlgebra[T] satisfy {
    generated_sigma_algebra(function(s: Set[T]) { T.is_open(s) }) = borel_sigma_algebra
}

/// The Borel sigma-algebra is the sigma-algebra generated by the open sets.
theorem borel_sigma_algebra_eq_generated_open[T: TopologicalSpace] {
    borel_sigma_algebra[T] =
        generated_sigma_algebra(function(s: Set[T]) { T.is_open(s) })
} by { }

/// Every open set is Borel-measurable.
theorem borel_sigma_algebra_open_is_measurable[T: TopologicalSpace](s: Set[T]) {
    T.is_open(s) implies borel_sigma_algebra[T].is_measurable(s)
} by {
    let g: Set[T] -> Bool = function(u: Set[T]) { T.is_open(u) }
    if T.is_open(s) {
        generated_sigma_algebra_contains_generator(g, s)
        borel_sigma_algebra[T].is_measurable(s)
    }
}

/// Every closed set is Borel-measurable.
theorem borel_sigma_algebra_closed_is_measurable[T: TopologicalSpace](s: Set[T]) {
    is_closed(s) implies borel_sigma_algebra[T].is_measurable(s)
} by {
    if is_closed(s) {
        borel_sigma_algebra_open_is_measurable[T](s.c)
        sigma_algebra_measurable_compl(borel_sigma_algebra[T], s.c)
        borel_sigma_algebra[T].is_measurable(s.c.c)
        compl_of_compl_is_self(s)
        borel_sigma_algebra[T].is_measurable(s)
    }
}

/// The Borel sigma-algebra is also generated by the closed sets.
theorem borel_sigma_algebra_eq_generated_closed[T: TopologicalSpace] {
    borel_sigma_algebra[T] =
        generated_sigma_algebra(function(s: Set[T]) { is_closed(s) })
} by {
    let go: Set[T] -> Bool = function(s: Set[T]) { T.is_open(s) }
    let gc: Set[T] -> Bool = function(s: Set[T]) { is_closed(s) }
    forall(s: Set[T]) {
        if gc(s) {
            borel_sigma_algebra_closed_is_measurable[T](s)
            borel_sigma_algebra[T].is_measurable(s)
            generated_sigma_algebra(go).is_measurable(s)
        }
    }
    generated_sigma_algebra_is_smallest(gc, generated_sigma_algebra(go))
    forall(s: Set[T]) {
        if go(s) {
            compl_of_compl_is_self(s)
            gc(s.c)
            generated_sigma_algebra_contains_generator(gc, s.c)
            sigma_algebra_measurable_compl(generated_sigma_algebra(gc), s.c)
            generated_sigma_algebra(gc).is_measurable(s.c.c)
            generated_sigma_algebra(gc).is_measurable(s)
        }
    }
    generated_sigma_algebra_is_smallest(go, generated_sigma_algebra(gc))
    sigma_algebra_le(generated_sigma_algebra(go), generated_sigma_algebra(gc))
    sigma_algebra_le_antisymm(generated_sigma_algebra(go), generated_sigma_algebra(gc))
    generated_sigma_algebra(go) = generated_sigma_algebra(gc)
}

/// A continuous function between topological spaces is measurable with
/// respect to the Borel sigma-algebras.
theorem continuous_is_borel_measurable[X: TopologicalSpace, Y: TopologicalSpace](f: X -> Y) {
    is_continuous(f) implies
        is_measurable_fn(f, borel_sigma_algebra[X], borel_sigma_algebra[Y])
} by {
    if is_continuous(f) {
        is_continuous(f) = forall(v: Set[Y]) {
            Y.is_open(v) implies X.is_open(set_preimage(f, v))
        }
        let gY: Set[Y] -> Bool = function(u: Set[Y]) { Y.is_open(u) }
        forall(t: Set[Y]) {
            if gY(t) {
                borel_sigma_algebra_open_is_measurable[X](set_preimage(f, t))
                borel_sigma_algebra[X].is_measurable(set_preimage(f, t))
            }
        }
        is_measurable_fn_into_generated(f, borel_sigma_algebra[X], gY)
        is_measurable_fn(f, borel_sigma_algebra[X], generated_sigma_algebra(gY))
        is_measurable_fn(f, borel_sigma_algebra[X], borel_sigma_algebra[Y])
    }
}

/// A countable union of open sets is Borel-measurable.
theorem borel_sigma_algebra_indexed_union_open_is_measurable[T: TopologicalSpace](
        family: Nat -> Set[T]) {
    (forall(n: Nat) { T.is_open(family(n)) })
        implies borel_sigma_algebra[T].is_measurable(indexed_union(family))
} by {
    if forall(n: Nat) { T.is_open(family(n)) } {
        forall(n: Nat) {
            borel_sigma_algebra_open_is_measurable[T](family(n))
            borel_sigma_algebra[T].is_measurable(family(n))
        }
        sigma_algebra_measurable_indexed_union(borel_sigma_algebra[T], family)
    }
}

/// A countable intersection of closed sets is Borel-measurable.
theorem borel_sigma_algebra_indexed_intersection_closed_is_measurable[T: TopologicalSpace](
        family: Nat -> Set[T]) {
    (forall(n: Nat) { is_closed(family(n)) })
        implies borel_sigma_algebra[T].is_measurable(indexed_intersection(family))
} by {
    if forall(n: Nat) { is_closed(family(n)) } {
        forall(n: Nat) {
            borel_sigma_algebra_closed_is_measurable[T](family(n))
            borel_sigma_algebra[T].is_measurable(family(n))
        }
        sigma_algebra_measurable_indexed_intersection(borel_sigma_algebra[T], family)
    }
}

/// Constant functions are Borel-measurable.
theorem borel_sigma_algebra_const_is_measurable[X: TopologicalSpace, Y: TopologicalSpace](c: Y) {
    is_measurable_fn(function(x: X) { c }, borel_sigma_algebra[X], borel_sigma_algebra[Y])
} by {
    is_measurable_fn_const(c, borel_sigma_algebra[X], borel_sigma_algebra[Y])
}

/// The identity function is Borel-measurable.
theorem borel_sigma_algebra_identity_is_measurable[T: TopologicalSpace] {
    is_measurable_fn(identity_fn[T], borel_sigma_algebra[T], borel_sigma_algebra[T])
} by {
    is_measurable_fn_identity(borel_sigma_algebra[T])
}

/// The indicator function of a Borel-measurable set is Borel-measurable into
/// the discrete sigma-algebra on `Bool`.
theorem borel_sigma_algebra_indicator_fn_is_measurable[T: TopologicalSpace](
        s: Set[T]) {
    borel_sigma_algebra[T].is_measurable(s) implies
        is_measurable_fn(indicator_fn(s),
            borel_sigma_algebra[T], SigmaAlgebra[Bool].discrete)
} by {
    if borel_sigma_algebra[T].is_measurable(s) {
        indicator_fn_is_measurable_fn(s, borel_sigma_algebra[T])
    }
}

/// The indicator function of an open set is Borel-measurable into the discrete
/// sigma-algebra on `Bool`.
theorem borel_sigma_algebra_indicator_fn_open[T: TopologicalSpace](s: Set[T]) {
    T.is_open(s) implies
        is_measurable_fn(indicator_fn(s),
            borel_sigma_algebra[T], SigmaAlgebra[Bool].discrete)
} by {
    if T.is_open(s) {
        borel_sigma_algebra_open_is_measurable[T](s)
        borel_sigma_algebra[T].is_measurable(s)
        indicator_fn_is_measurable_fn(s, borel_sigma_algebra[T])
    }
}

/// The indicator function of a closed set is Borel-measurable into the discrete
/// sigma-algebra on `Bool`.
theorem borel_sigma_algebra_indicator_fn_closed[T: TopologicalSpace](s: Set[T]) {
    is_closed(s) implies
        is_measurable_fn(indicator_fn(s),
            borel_sigma_algebra[T], SigmaAlgebra[Bool].discrete)
} by {
    if is_closed(s) {
        borel_sigma_algebra_closed_is_measurable[T](s)
        borel_sigma_algebra[T].is_measurable(s)
        indicator_fn_is_measurable_fn(s, borel_sigma_algebra[T])
    }
}

/// If the indicator function of `s` is Borel-measurable into the discrete
/// sigma-algebra on `Bool`, then `s` is Borel-measurable.
theorem borel_set_measurable_of_indicator_fn_measurable[T: TopologicalSpace](
        s: Set[T]) {
    is_measurable_fn(indicator_fn(s),
        borel_sigma_algebra[T], SigmaAlgebra[Bool].discrete)
        implies borel_sigma_algebra[T].is_measurable(s)
} by {
    if is_measurable_fn(indicator_fn(s),
        borel_sigma_algebra[T], SigmaAlgebra[Bool].discrete) {
        set_measurable_of_indicator_fn_measurable(s, borel_sigma_algebra[T])
    }
}

/// The indicator function of `s` is Borel-measurable into the discrete
/// sigma-algebra on `Bool` if and only if `s` is Borel-measurable.
theorem borel_sigma_algebra_indicator_fn_iff[T: TopologicalSpace](s: Set[T]) {
    is_measurable_fn(indicator_fn(s),
        borel_sigma_algebra[T], SigmaAlgebra[Bool].discrete) iff
        borel_sigma_algebra[T].is_measurable(s)
} by {
    indicator_fn_is_measurable_iff(s, borel_sigma_algebra[T])
}

/// The preimage sigma-algebra of the Borel sigma-algebra under any function
/// is generated by the preimages of the open sets.
theorem preimage_sigma_algebra_borel_eq[X: TopologicalSpace, Y: TopologicalSpace](
        f: X -> Y) {
    preimage_sigma_algebra(f, borel_sigma_algebra[Y]) =
        generated_sigma_algebra(
            preimage_generator(f, function(s: Set[Y]) { Y.is_open(s) }))
} by {
    let g: Set[Y] -> Bool = function(s: Set[Y]) { Y.is_open(s) }
    preimage_sigma_algebra_generated_eq(f, g)
}

/// Every set open in the subspace `a` is Borel-measurable in the subspace.
theorem borel_sigma_algebra_open_in_subspace_is_measurable[T: TopologicalSpace](
        a: Set[T], u: Set[T]) {
    is_open_in_subspace[T](a, u) implies
        is_measurable_in_subspace(borel_sigma_algebra[T], a, u)
} by {
    if is_open_in_subspace[T](a, u) {
        let v: Set[T] satisfy {
            T.is_open(v) and u = v.intersection(a)
        }
        borel_sigma_algebra_open_is_measurable[T](v)
        exists(w: Set[T]) {
            borel_sigma_algebra[T].is_measurable(w) and u = w.intersection(a)
        }
    }
}

/// Every set closed in the subspace `a` is Borel-measurable in the subspace.
theorem borel_sigma_algebra_closed_in_subspace_is_measurable[T: TopologicalSpace](
        a: Set[T], u: Set[T]) {
    is_closed_in_subspace[T](a, u) implies
        is_measurable_in_subspace(borel_sigma_algebra[T], a, u)
} by {
    if is_closed_in_subspace[T](a, u) {
        let c: Set[T] satisfy {
            is_closed[T](c) and u = c.intersection(a)
        }
        borel_sigma_algebra_closed_is_measurable[T](c)
        exists(w: Set[T]) {
            borel_sigma_algebra[T].is_measurable(w) and u = w.intersection(a)
        }
    }
}

/// The indicator function of a set open in the subspace `a` is measurable from
/// the trace Borel sigma-algebra into the discrete sigma-algebra on `Bool`.
theorem borel_subspace_indicator_fn_open[T: TopologicalSpace](a: Set[T], u: Set[T]) {
    is_open_in_subspace[T](a, u) implies
        is_measurable_fn(indicator_fn(u),
            trace_sigma_algebra(borel_sigma_algebra[T], a),
            SigmaAlgebra[Bool].discrete)
} by {
    if is_open_in_subspace[T](a, u) {
        borel_sigma_algebra_open_in_subspace_is_measurable[T](a, u)
        is_measurable_in_subspace(borel_sigma_algebra[T], a, u)
        subspace_indicator_fn_is_measurable(borel_sigma_algebra[T], a, u)
    }
}

/// The indicator function of a set closed in the subspace `a` is measurable from
/// the trace Borel sigma-algebra into the discrete sigma-algebra on `Bool`.
theorem borel_subspace_indicator_fn_closed[T: TopologicalSpace](a: Set[T], u: Set[T]) {
    is_closed_in_subspace[T](a, u) implies
        is_measurable_fn(indicator_fn(u),
            trace_sigma_algebra(borel_sigma_algebra[T], a),
            SigmaAlgebra[Bool].discrete)
} by {
    if is_closed_in_subspace[T](a, u) {
        borel_sigma_algebra_closed_in_subspace_is_measurable[T](a, u)
        is_measurable_in_subspace(borel_sigma_algebra[T], a, u)
        subspace_indicator_fn_is_measurable(borel_sigma_algebra[T], a, u)
    }
}

/// The composition of two Borel-measurable functions is Borel-measurable.
theorem borel_sigma_algebra_compose_is_measurable[X: TopologicalSpace, Y: TopologicalSpace, Z: TopologicalSpace](
        f: Y -> Z, g: X -> Y) {
    is_measurable_fn(g, borel_sigma_algebra[X], borel_sigma_algebra[Y])
        and is_measurable_fn(f, borel_sigma_algebra[Y], borel_sigma_algebra[Z])
        implies is_measurable_fn(compose(f, g), borel_sigma_algebra[X], borel_sigma_algebra[Z])
} by {
    is_measurable_fn_compose(f, g,
        borel_sigma_algebra[X], borel_sigma_algebra[Y], borel_sigma_algebra[Z])
}

/// A Borel-measurable function pulls back Borel-measurable sets to Borel-measurable sets.
theorem borel_measurable_preimage_measurable[X: TopologicalSpace, Y: TopologicalSpace](
        f: X -> Y, t: Set[Y]) {
    is_measurable_fn(f, borel_sigma_algebra[X], borel_sigma_algebra[Y])
        and borel_sigma_algebra[Y].is_measurable(t)
        implies borel_sigma_algebra[X].is_measurable(set_preimage(f, t))
} by {
    if is_measurable_fn(f, borel_sigma_algebra[X], borel_sigma_algebra[Y])
            and borel_sigma_algebra[Y].is_measurable(t) {
        is_measurable_fn_preimage_measurable(
            f, borel_sigma_algebra[X], borel_sigma_algebra[Y], t)
    }
}

/// A continuous map pulls back Borel-measurable sets to Borel-measurable sets.
theorem continuous_preimage_borel_measurable[X: TopologicalSpace, Y: TopologicalSpace](
        f: X -> Y, t: Set[Y]) {
    is_continuous(f) and borel_sigma_algebra[Y].is_measurable(t)
        implies borel_sigma_algebra[X].is_measurable(set_preimage(f, t))
} by {
    if is_continuous(f) and borel_sigma_algebra[Y].is_measurable(t) {
        continuous_is_borel_measurable(f)
        is_measurable_fn(f, borel_sigma_algebra[X], borel_sigma_algebra[Y])
        borel_measurable_preimage_measurable(f, t)
    }
}

/// A subspace-open set is measurable in the trace Borel sigma-algebra.
theorem trace_borel_open_in_subspace_is_measurable[T: TopologicalSpace](
        a: Set[T], u: Set[T]) {
    is_open_in_subspace[T](a, u) implies
        trace_sigma_algebra(borel_sigma_algebra[T], a).is_measurable(u)
} by {
    if is_open_in_subspace[T](a, u) {
        borel_sigma_algebra_open_in_subspace_is_measurable[T](a, u)
        is_measurable_in_subspace(borel_sigma_algebra[T], a, u)
        trace_sigma_algebra_measurable_of_subspace(borel_sigma_algebra[T], a, u)
    }
}

/// A subspace-closed set is measurable in the trace Borel sigma-algebra.
theorem trace_borel_closed_in_subspace_is_measurable[T: TopologicalSpace](
        a: Set[T], u: Set[T]) {
    is_closed_in_subspace[T](a, u) implies
        trace_sigma_algebra(borel_sigma_algebra[T], a).is_measurable(u)
} by {
    if is_closed_in_subspace[T](a, u) {
        borel_sigma_algebra_closed_in_subspace_is_measurable[T](a, u)
        is_measurable_in_subspace(borel_sigma_algebra[T], a, u)
        trace_sigma_algebra_measurable_of_subspace(borel_sigma_algebra[T], a, u)
    }
}
