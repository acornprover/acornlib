/// The Lebesgue outer measure on the real line, built from countable covers
/// by closed intervals, and the Lebesgue measurable sets via Carathéodory's
/// criterion.
///
/// The library's reals have no element `infinity`, so the outer measure is
/// defined as a total function that agrees with the infimum of the cost set
/// whenever that infimum exists, and takes the value `0` otherwise (e.g. on
/// unbounded sets, whose countable interval covers never have finite total
/// length).  All theorems below that assert a value or an inequality for the
/// outer measure are therefore stated under the hypothesis that the relevant
/// cost sets have infima.
///
/// The cost of a countable interval cover is represented through the partial
/// sums of the interval lengths, in the same spirit as the finite form of
/// countable subadditivity used in `measure.ac`: a cover contributes the
/// real `L` to the cost set exactly when its partial sums converge to `L`.
from data.basic.set import Set, indexed_union, indexed_union_contains_eq,
    indexed_union_contains_of_contains, indexed_union_contains_witness,
    set_ext, subset_trans, union_contains_eq, union_contains_left,
    union_contains_cases, intersection_contains_eq, difference_contains_eq,
    compl_contains_eq, empty_set_contains_eq, universal_set_contains_eq,
    empty_set_is_always_subset, intersection_with_universal_is_self,
    intersection_with_empty_is_empty, union_comm, compl_of_compl_is_self
from data.basic.witness import choose_or_default, choose_or_default_spec
from nat import Nat, zero_or_suc, alt_induction, alt_suc_ne_zero, only_zero_lte_zero
from order import lte_refl, lte_antisymm, lt_of_lte_of_lt, lt_irrefl,
    not_lt_imp_gte, closed_interval
from order_set import closed_interval_set, closed_interval_set_contains_eq
from real import Real, lte_trans, lte_trans_eq, lte_add_left, lte_add_right,
    close_imp_bounds, self_close, lt_neg_swap_neg, gt_zero_imp_pos,
    add_neg_eq_zero, neg_neg, is_lower_bound, partial_nonneg, converges_to,
    tail_bound, tail_bound_implies_is_close, const_converges_to, is_nonempty,
    has_upper_bound, is_set_lower_bound, is_set_upper_bound, is_set_infimum,
    is_set_supremum, completeness, eventual_eq
from list import partial, partial_zero, partial_one, partial_split_last,
    partial_pointwise_eq
from analysis.topology.measure import difference_universal_is_empty,
    difference_empty_is_self, intersection_compl_is_difference,
    difference_compl_is_intersection, difference_iterated_union,
    intersection_union_intersect_left, intersection_union_difference_left,
    union_compl_is_intersection_compl, difference_eq_intersection_compl
from analysis.topology.measurable_space import has_measurable_empty_constraint,
    closed_under_compl_constraint, closed_under_countable_union_constraint

numerals Nat
numerals Real

/// The length of the interval with endpoints `a` and `b`.
define interval_length(a: Real, b: Real) -> Real {
    b - a
}

/// True if the interval with endpoints `a` and `b` is proper, i.e. has
/// nonnegative length.
define is_proper_interval(a: Real, b: Real) -> Bool {
    a <= b
}

/// The countable family of closed intervals given by the endpoint sequences
/// `a` and `b`.
define interval_family(a: Nat -> Real, b: Nat -> Real) -> Nat -> Set[Real] {
    function(n: Nat) { closed_interval_set(a(n), b(n)) }
}

/// The union of the closed intervals of a countable family.
define interval_family_union(a: Nat -> Real, b: Nat -> Real) -> Set[Real] {
    indexed_union(interval_family(a, b))
}

/// The sequence of partial sums of the lengths of the intervals of a family.
define interval_cost_seq(a: Nat -> Real, b: Nat -> Real) -> Nat -> Real {
    function(n: Nat) {
        partial(function(k: Nat) { interval_length(a(k), b(k)) }, n)
    }
}

/// True if `L` is the total cost of some proper countable interval cover
/// of `A`: a sequence of proper closed intervals whose union contains `A`
/// and whose partial length sums converge to `L`.
define is_interval_cover_cost(s: Set[Real], c: Real) -> Bool {
    exists(a: Nat -> Real, b: Nat -> Real) {
        (forall(n: Nat) { is_proper_interval(a(n), b(n)) })
            and s.subset(interval_family_union(a, b))
            and converges_to(interval_cost_seq(a, b), c)
    }
}

/// The set of costs of the countable interval covers of `A`.
define lebesgue_cost_set(s: Set[Real]) -> Set[Real] {
    Set[Real].new(function(c: Real) { is_interval_cover_cost(s, c) })
}

/// Membership in the cost set is the cover-cost predicate.
theorem lebesgue_cost_set_contains_eq(s: Set[Real], c: Real) {
    lebesgue_cost_set(s).contains(c) = is_interval_cover_cost(s, c)
} by {
}

/// True if the cost set of `A` has an infimum.
define lebesgue_cost_has_infimum(s: Set[Real]) -> Bool {
    exists(v: Real) {
        is_set_infimum(lebesgue_cost_set(s), v)
    }
}

/// The Lebesgue outer measure of `A`: the infimum of the costs of the
/// countable interval covers of `A` when that infimum exists, and `0`
/// otherwise.
define lebesgue_outer_measure(s: Set[Real]) -> Real {
    choose_or_default(function(v: Real) { is_set_infimum(lebesgue_cost_set(s), v) }, Real.0)
}

/// The sequence that is `x` at index zero and zero after that.
define delta_seq(x: Real, k: Nat) -> Real {
    if k = Nat.0 { x } else { Real.0 }
}

/// The partial sum of a sequence that is `x` at index zero and zero
/// afterwards equals `x` from index one onward.
theorem partial_delta_seq(x: Real, n: Nat) {
    partial(delta_seq(x), n.suc) = x
} by {
    define p(k: Nat) -> Bool {
        partial(delta_seq(x), k.suc) = x
    }
    partial_one(delta_seq(x))
    partial(delta_seq(x), Nat.1) = delta_seq(x, Nat.0)
    delta_seq(x, Nat.0) = x
    partial(delta_seq(x), Nat.1) = x
    p(Nat.0)
    forall(k: Nat) {
        if p(k) {
            partial_split_last(delta_seq(x), k.suc)
            partial(delta_seq(x), k.suc.suc) = partial(delta_seq(x), k.suc) + delta_seq(x, k.suc)
            partial(delta_seq(x), k.suc) = x
            if k.suc = Nat.0 {
                alt_suc_ne_zero(k)
                false
            }
            if k.suc != Nat.0 {
                delta_seq(x, k.suc) = Real.0
            }
            delta_seq(x, k.suc) = Real.0
            x + Real.0 = x
            partial(delta_seq(x), k.suc.suc) = x
            p(k.suc)
        }
    }
    p(Nat.0) and forall(k: Nat) { p(k) implies p(k.suc) }
    alt_induction(p)
    forall(k: Nat) { p(k) }
    p(n)
}

/// The partial sums of the delta sequence equal `x` from index one onward.
theorem partial_delta_seq_later(x: Real, i: Nat) {
    Nat.1 <= i implies partial(delta_seq(x), i) = x
} by {
    if Nat.1 <= i {
        zero_or_suc(i)
        if i = Nat.0 {
            only_zero_lte_zero(Nat.1)
            Nat.1 = Nat.0
            alt_suc_ne_zero(Nat.0)
            false
        }
        if exists(j: Nat) { i = j.suc } {
            let j: Nat satisfy { i = j.suc }
            partial_delta_seq(x, j)
            partial(delta_seq(x), j.suc) = x
            i = j.suc
            partial(delta_seq(x), i) = x
        }
        partial(delta_seq(x), i) = x
    }
}

/// The partial sums of the constant zero sequence are all zero.
theorem partial_const_zero(n: Nat) {
    partial(function(k: Nat) { Real.0 }, n) = Real.0
} by {
    define p(k: Nat) -> Bool {
        partial(function(j: Nat) { Real.0 }, k) = Real.0
    }
    partial_zero(function(j: Nat) { Real.0 })
    partial(function(j: Nat) { Real.0 }, Nat.0) = Real.0
    p(Nat.0)
    forall(k: Nat) {
        if p(k) {
            partial_split_last(function(j: Nat) { Real.0 }, k)
            partial(function(j: Nat) { Real.0 }, k.suc) = partial(function(j: Nat) { Real.0 }, k) + function(j: Nat) { Real.0 }(k)
            partial(function(j: Nat) { Real.0 }, k) = Real.0
            function(j: Nat) { Real.0 }(k) = Real.0
            Real.0 + Real.0 = Real.0
            partial(function(j: Nat) { Real.0 }, k.suc) = Real.0
            p(k.suc)
        }
    }
    p(Nat.0) and forall(k: Nat) { p(k) implies p(k.suc) }
    alt_induction(p)
    forall(k: Nat) { p(k) }
    p(n)
}

/// Pointwise equal sequences have the same limits.
theorem converges_to_of_pointwise_eq(a: Nat -> Real, b: Nat -> Real, c: Real) {
    (forall(n: Nat) { a(n) = b(n) }) and converges_to(a, c)
        implies converges_to(b, c)
} by {
    if forall(n: Nat) { a(n) = b(n) } and converges_to(a, c) {
        forall(eps: Real) {
            if eps.is_positive {
                converges_to(a, c) = forall(e: Real) {
                    e.is_positive implies exists(n: Nat) { tail_bound(a, c, n, e) }
                }
                exists(n: Nat) { tail_bound(a, c, n, eps) }
                let n: Nat satisfy { tail_bound(a, c, n, eps) }
                forall(i: Nat) {
                    if n <= i {
                        tail_bound_implies_is_close(a, c, n, eps, i)
                        a(i).is_close(c, eps)
                        a(i) = b(i)
                        b(i).is_close(c, eps)
                    }
                }
                tail_bound(b, c, n, eps)
                exists(m: Nat) { tail_bound(b, c, m, eps) }
            }
        }
        converges_to(b, c)
    }
}

/// A sequence of nonnegative terms converges to a nonnegative limit.
theorem converges_to_nonneg_imp_limit_nonneg(q: Nat -> Real, c: Real) {
    converges_to(q, c) and (forall(n: Nat) { Real.0 <= q(n) })
        implies Real.0 <= c
} by {
    if converges_to(q, c) and forall(n: Nat) { Real.0 <= q(n) } {
        if c < Real.0 {
            let eps = -c
            lt_neg_swap_neg(c, Real.0)
            Real.0 < -c
            Real.0 < eps
            gt_zero_imp_pos(eps)
            eps.is_positive
            converges_to(q, c) = forall(e: Real) {
                e.is_positive implies exists(n: Nat) { tail_bound(q, c, n, e) }
            }
            exists(n: Nat) { tail_bound(q, c, n, eps) }
            let n: Nat satisfy { tail_bound(q, c, n, eps) }
            tail_bound_implies_is_close(q, c, n, eps, n)
            q(n).is_close(c, eps)
            close_imp_bounds(q(n), c, eps)
            q(n) < c + eps
            c + eps = c + -c
            add_neg_eq_zero(c)
            c + -c = Real.0
            c + eps = Real.0
            q(n) < Real.0
            lt_of_lte_of_lt(Real.0, q(n), Real.0)
            Real.0 < Real.0
            lt_irrefl(Real.0)
            false
        }
        not_lt_imp_gte(c, Real.0)
        Real.0 <= c
    }
}

/// A non-strict inequality gives a nonnegative difference.
theorem lte_imp_sub_nonneg(a: Real, b: Real) {
    a <= b implies Real.0 <= b - a
} by {
    if a <= b {
        lte_add_right(a, b, -a)
        a + -a <= b + -a
        add_neg_eq_zero(a)
        a + -a = Real.0
        Real.0 <= b + -a
        b + -a = b - a
        Real.0 <= b - a
    }
}

/// Every cost of a countable interval cover of `s` is nonnegative.
theorem lebesgue_cost_nonneg(s: Set[Real], c: Real) {
    lebesgue_cost_set(s).contains(c) implies Real.0 <= c
} by {
    if lebesgue_cost_set(s).contains(c) {
        lebesgue_cost_set_contains_eq(s, c)
        is_interval_cover_cost(s, c)
        let (a: Nat -> Real, b: Nat -> Real) satisfy {
            (forall(n: Nat) { is_proper_interval(a(n), b(n)) })
                and s.subset(interval_family_union(a, b))
                and converges_to(interval_cost_seq(a, b), c)
        }
        forall(k: Nat) {
            is_proper_interval(a(k), b(k))
            lte_imp_sub_nonneg(a(k), b(k))
            Real.0 <= b(k) - a(k)
            interval_length(a(k), b(k)) = b(k) - a(k)
            Real.0 <= interval_length(a(k), b(k))
        }
        forall(n: Nat) {
            forall(i: Nat) {
                Real.0 <= interval_length(a(i), b(i))
                function(k: Nat) { interval_length(a(k), b(k)) }(i) = interval_length(a(i), b(i))
                Real.0 <= function(k: Nat) { interval_length(a(k), b(k)) }(i)
            }
            is_lower_bound(function(k: Nat) { interval_length(a(k), b(k)) }, Real.0) = forall(i: Nat) {
                Real.0 <= function(k: Nat) { interval_length(a(k), b(k)) }(i)
            }
            is_lower_bound(function(k: Nat) { interval_length(a(k), b(k)) }, Real.0)
            partial_nonneg(function(k: Nat) { interval_length(a(k), b(k)) }, n)
            partial(function(k: Nat) { interval_length(a(k), b(k)) }, n) >= Real.0
            Real.0 <= partial(function(k: Nat) { interval_length(a(k), b(k)) }, n)
            interval_cost_seq(a, b, n) = partial(function(k: Nat) { interval_length(a(k), b(k)) }, n)
            Real.0 <= interval_cost_seq(a, b, n)
        }
        converges_to_nonneg_imp_limit_nonneg(interval_cost_seq(a, b), c)
        Real.0 <= c
    }
}

/// Negating both sides of a non-strict inequality reverses it.
theorem neg_lte_swap(a: Real, b: Real) {
    a <= b implies -b <= -a
} by {
    if a <= b {
        lte_imp_sub_nonneg(a, b)
        Real.0 <= b - a
        b - a = -a + b
        Real.0 <= -a + b
        lte_add_left(Real.0, -a + b, -b)
        Real.0 + -b <= -b + (-a + b)
        Real.0 + -b = -b
        -b + (-a + b) = -b + (b + -a)
        -b + (b + -a) = (-b + b) + -a
        -b + b = Real.0
        (-b + b) + -a = Real.0 + -a
        Real.0 + -a = -a
        -b + (-a + b) = -a
        -b <= -a
    }
}

/// True if `x` is the negation of a member of `s`.
define neg_set_contains(s: Set[Real], x: Real) -> Bool {
    exists(a: Real) {
        s.contains(a) and x = -a
    }
}

/// The negation of a set `s` is the set of the negations of its members.
define neg_set(s: Set[Real]) -> Set[Real] {
    Set[Real].new(neg_set_contains(s))
}

/// Membership in a negated set is the negated-membership predicate.
theorem neg_set_contains_eq(s: Set[Real], x: Real) {
    neg_set(s).contains(x) = neg_set_contains(s, x)
} by {
}

/// The negation of a member of `s` lies in the negated set.
theorem neg_set_membership(s: Set[Real], a: Real) {
    s.contains(a) implies neg_set(s).contains(-a)
} by {
    if s.contains(a) {
        neg_set_contains_eq(s, -a)
        neg_set_contains(s, -a)
        exists(b: Real) { s.contains(b) and -a = -b }
        neg_set(s).contains(-a)
    }
}

/// A member of a negated set is the negation of a member of the set.
theorem neg_set_witness(s: Set[Real], x: Real) {
    neg_set(s).contains(x) implies exists(a: Real) { s.contains(a) and x = -a }
} by {
    if neg_set(s).contains(x) {
        neg_set_contains_eq(s, x)
        exists(a: Real) { s.contains(a) and x = -a }
    }
}

/// Negating a set twice returns the set itself.
theorem neg_set_involutive(s: Set[Real]) {
    neg_set(neg_set(s)) = s
} by {
    forall(x: Real) {
        if neg_set(neg_set(s)).contains(x) {
            neg_set_witness(neg_set(s), x)
            let a: Real satisfy { neg_set(s).contains(a) and x = -a }
            neg_set_witness(s, a)
            let b: Real satisfy { s.contains(b) and a = -b }
            x = -a
            a = -b
            x = -(-b)
            neg_neg(b)
            -(-b) = b
            x = b
            s.contains(b)
            s.contains(x)
        }
        if s.contains(x) {
            neg_set_membership(s, x)
            neg_set(s).contains(-x)
            neg_set_membership(neg_set(s), -x)
            neg_set(neg_set(s)).contains(-(-x))
            neg_neg(x)
            -(-x) = x
            neg_set(neg_set(s)).contains(x)
        }
        neg_set(neg_set(s)).contains(x) = s.contains(x)
    }
    set_ext(neg_set(neg_set(s)), s)
}

/// A supremum of the negated set gives an infimum of the set.
theorem neg_set_supremum_imp_infimum(s: Set[Real], sup_n: Real) {
    is_set_supremum(neg_set(s), sup_n) implies is_set_infimum(s, -sup_n)
} by {
    if is_set_supremum(neg_set(s), sup_n) {
        forall(a: Real) {
            if s.contains(a) {
                neg_set_membership(s, a)
                neg_set(s).contains(-a)
                is_set_supremum(neg_set(s), sup_n) =
                    (is_set_upper_bound(neg_set(s), sup_n)
                        and forall(b: Real) {
                            is_set_upper_bound(neg_set(s), b) implies sup_n <= b
                        })
                is_set_upper_bound(neg_set(s), sup_n)
                is_set_upper_bound(neg_set(s), sup_n) = forall(x: Real) {
                    neg_set(s).contains(x) implies x <= sup_n
                }
                neg_set(s).contains(-a) implies -a <= sup_n
                -a <= sup_n
                neg_lte_swap(-a, sup_n)
                -sup_n <= -(-a)
                neg_neg(a)
                -(-a) = a
                -sup_n <= a
            }
        }
        is_set_lower_bound(s, -sup_n)
        forall(lb: Real) {
            if is_set_lower_bound(s, lb) {
                forall(x: Real) {
                    if neg_set(s).contains(x) {
                        neg_set_witness(s, x)
                        let c: Real satisfy { s.contains(c) and x = -c }
                        is_set_lower_bound(s, lb) = forall(y: Real) {
                            s.contains(y) implies lb <= y
                        }
                        lb <= c
                        neg_lte_swap(lb, c)
                        -c <= -lb
                        x = -c
                        x <= -lb
                    }
                }
                is_set_upper_bound(neg_set(s), -lb)
                is_set_supremum(neg_set(s), sup_n) =
                    (is_set_upper_bound(neg_set(s), sup_n)
                        and forall(b: Real) {
                            is_set_upper_bound(neg_set(s), b) implies sup_n <= b
                        })
                forall(b: Real) {
                    is_set_upper_bound(neg_set(s), b) implies sup_n <= b
                }
                is_set_upper_bound(neg_set(s), -lb) implies sup_n <= -lb
                sup_n <= -lb
                neg_lte_swap(sup_n, -lb)
                -(-lb) <= -sup_n
                neg_neg(lb)
                -(-lb) = lb
                lb <= -sup_n
            }
        }
        is_set_infimum(s, -sup_n)
    }
}

/// An infimum of a set is unique.
theorem set_infimum_unique(s: Set[Real], v1: Real, v2: Real) {
    is_set_infimum(s, v1) and is_set_infimum(s, v2) implies v1 = v2
} by {
    if is_set_infimum(s, v1) and is_set_infimum(s, v2) {
        is_set_infimum(s, v1) = (is_set_lower_bound(s, v1) and forall(b: Real) {
            is_set_lower_bound(s, b) implies b <= v1
        })
        is_set_infimum(s, v2) = (is_set_lower_bound(s, v2) and forall(b: Real) {
            is_set_lower_bound(s, b) implies b <= v2
        })
        is_set_lower_bound(s, v1)
        is_set_lower_bound(s, v2)
        is_set_lower_bound(s, v2) implies v1 <= v2
        is_set_lower_bound(s, v1) implies v2 <= v1
        v1 <= v2
        v2 <= v1
        lte_antisymm(v1, v2)
        v1 = v2
    }
}

/// Every nonempty set of real numbers bounded below has an infimum.
theorem infimum_exists_of_nonempty_bounded_below(s: Set[Real]) {
    is_nonempty(s) and exists(lb: Real) { is_set_lower_bound(s, lb) }
        implies exists(v: Real) { is_set_infimum(s, v) }
} by {
    if is_nonempty(s) and exists(lb: Real) { is_set_lower_bound(s, lb) } {
        let lb: Real satisfy { is_set_lower_bound(s, lb) }
        let a: Real satisfy { s.contains(a) }
        neg_set_membership(s, a)
        neg_set(s).contains(-a)
        is_nonempty(neg_set(s))
        forall(x: Real) {
            if neg_set(s).contains(x) {
                neg_set_witness(s, x)
                let c: Real satisfy { s.contains(c) and x = -c }
                is_set_lower_bound(s, lb) = forall(y: Real) {
                    s.contains(y) implies lb <= y
                }
                lb <= c
                neg_lte_swap(lb, c)
                -c <= -lb
                x = -c
                x <= -lb
            }
        }
        is_set_upper_bound(neg_set(s), -lb)
        has_upper_bound(neg_set(s))
        completeness(neg_set(s))
        is_nonempty(neg_set(s)) and has_upper_bound(neg_set(s))
        exists(sup_n: Real) { is_set_supremum(neg_set(s), sup_n) }
        let sup_n: Real satisfy { is_set_supremum(neg_set(s), sup_n) }
        neg_set_supremum_imp_infimum(s, sup_n)
        is_set_infimum(s, -sup_n)
        exists(v: Real) { is_set_infimum(s, v) }
    }
}

/// The outer measure of `A` is the infimum of the cost set of `A` whenever
/// that infimum exists.
theorem lebesgue_outer_measure_is_infimum(s: Set[Real]) {
    lebesgue_cost_has_infimum(s) implies
        is_set_infimum(lebesgue_cost_set(s), lebesgue_outer_measure(s))
} by {
    if lebesgue_cost_has_infimum(s) {
        lebesgue_cost_has_infimum(s) = exists(v: Real) {
            is_set_infimum(lebesgue_cost_set(s), v)
        }
        exists(v: Real) { is_set_infimum(lebesgue_cost_set(s), v) }
        choose_or_default_spec(function(v: Real) { is_set_infimum(lebesgue_cost_set(s), v) }, Real.0)
        is_set_infimum(lebesgue_cost_set(s),
            choose_or_default(function(v: Real) { is_set_infimum(lebesgue_cost_set(s), v) }, Real.0))
        lebesgue_outer_measure(s) =
            choose_or_default(function(v: Real) { is_set_infimum(lebesgue_cost_set(s), v) }, Real.0)
        is_set_infimum(lebesgue_cost_set(s), lebesgue_outer_measure(s))
    }
}

/// The lower endpoint of the empty cover.
define zero_interval_lower(n: Nat) -> Real {
    Real.0
}

/// The upper endpoint of the empty cover.
define zero_interval_upper(n: Nat) -> Real {
    Real.0
}

/// A sequence that is eventually constant at `a` converges to `a`.
theorem eventual_eq_imp_converges_to(q: Nat -> Real, a: Real) {
    eventual_eq(q, a) implies converges_to(q, a)
} by {
    if eventual_eq(q, a) {
        eventual_eq(q, a) = exists(n: Nat) {
            forall(i: Nat) { n <= i implies q(i) = a }
        }
        let n: Nat satisfy {
            forall(i: Nat) { n <= i implies q(i) = a }
        }
        forall(eps: Real) {
            if eps.is_positive {
                forall(i: Nat) {
                    if n <= i {
                        q(i) = a
                        self_close(a, eps)
                        q(i).is_close(a, eps)
                    }
                }
                tail_bound(q, a, n, eps)
            }
        }
        converges_to(q, a)
    }
}

/// The constant-zero sequence converges to zero.
theorem const_zero_converges_to_zero {
    converges_to(function(n: Nat) { Real.0 }, Real.0)
} by {
    forall(i: Nat) {
        if Nat.0 <= i {
            function(m: Nat) { Real.0 }(i) = Real.0
        }
    }
    eventual_eq(function(m: Nat) { Real.0 }, Real.0)
    eventual_eq_imp_converges_to(function(m: Nat) { Real.0 }, Real.0)
    converges_to(function(n: Nat) { Real.0 }, Real.0)
}

/// The cost sequence of the empty cover converges to zero.
theorem empty_cover_cost_converges {
    converges_to(interval_cost_seq(zero_interval_lower, zero_interval_upper), Real.0)
} by {
    forall(n: Nat) {
        interval_cost_seq(zero_interval_lower, zero_interval_upper, n) =
            partial(function(k: Nat) { interval_length(zero_interval_lower(k), zero_interval_upper(k)) }, n)
        forall(k: Nat) {
            interval_length(zero_interval_lower(k), zero_interval_upper(k)) = zero_interval_upper(k) - zero_interval_lower(k)
            zero_interval_upper(k) = Real.0
            zero_interval_lower(k) = Real.0
            zero_interval_upper(k) - zero_interval_lower(k) = Real.0
            interval_length(zero_interval_lower(k), zero_interval_upper(k)) = Real.0
        }
        partial(function(k: Nat) { interval_length(zero_interval_lower(k), zero_interval_upper(k)) }, n) =
            partial(function(k: Nat) { Real.0 }, n)
        partial(function(k: Nat) { Real.0 }, n) = Real.0
        interval_cost_seq(zero_interval_lower, zero_interval_upper, n) = Real.0
        function(m: Nat) { Real.0 }(n) = Real.0
        interval_cost_seq(zero_interval_lower, zero_interval_upper, n) = function(m: Nat) { Real.0 }(n)
    }
    const_zero_converges_to_zero
    converges_to(function(n: Nat) { Real.0 }, Real.0)
    converges_to_of_pointwise_eq(function(n: Nat) { Real.0 }, interval_cost_seq(zero_interval_lower, zero_interval_upper), Real.0)
    converges_to(interval_cost_seq(zero_interval_lower, zero_interval_upper), Real.0)
}

/// The constant-zero cover covers the empty set with cost zero.
theorem empty_cover_is_cost_zero {
    is_interval_cover_cost(Set[Real].empty_set, Real.0)
} by {
    forall(n: Nat) {
        is_proper_interval(zero_interval_lower(n), zero_interval_upper(n)) =
            (zero_interval_lower(n) <= zero_interval_upper(n))
        zero_interval_lower(n) = Real.0
        zero_interval_upper(n) = Real.0
        lte_refl(Real.0)
        Real.0 <= Real.0
        is_proper_interval(zero_interval_lower(n), zero_interval_upper(n))
    }
    empty_set_is_always_subset(interval_family_union(zero_interval_lower, zero_interval_upper))
    Set[Real].empty_set.subset(interval_family_union(zero_interval_lower, zero_interval_upper))
    empty_cover_cost_converges
    converges_to(interval_cost_seq(zero_interval_lower, zero_interval_upper), Real.0)
    exists(a: Nat -> Real, b: Nat -> Real) {
        (forall(n: Nat) { is_proper_interval(a(n), b(n)) })
            and Set[Real].empty_set.subset(interval_family_union(a, b))
            and converges_to(interval_cost_seq(a, b), Real.0)
    }
    is_interval_cover_cost(Set[Real].empty_set, Real.0)
}

/// Zero is the infimum of the cost set of the empty set.
theorem empty_cost_set_infimum_zero {
    is_set_infimum(lebesgue_cost_set(Set[Real].empty_set), Real.0)
} by {
    forall(c: Real) {
        if lebesgue_cost_set(Set[Real].empty_set).contains(c) {
            lebesgue_cost_nonneg(Set[Real].empty_set, c)
            Real.0 <= c
        }
    }
    is_set_lower_bound(lebesgue_cost_set(Set[Real].empty_set), Real.0)
    lebesgue_cost_set_contains_eq(Set[Real].empty_set, Real.0)
    empty_cover_is_cost_zero
    is_interval_cover_cost(Set[Real].empty_set, Real.0)
    lebesgue_cost_set(Set[Real].empty_set).contains(Real.0)
    forall(b: Real) {
        if is_set_lower_bound(lebesgue_cost_set(Set[Real].empty_set), b) {
            is_set_lower_bound(lebesgue_cost_set(Set[Real].empty_set), b) = forall(x: Real) {
                lebesgue_cost_set(Set[Real].empty_set).contains(x) implies b <= x
            }
            lebesgue_cost_set(Set[Real].empty_set).contains(Real.0) implies b <= Real.0
            b <= Real.0
        }
    }
    is_set_infimum(lebesgue_cost_set(Set[Real].empty_set), Real.0)
}

/// The Lebesgue outer measure of the empty set is zero.
theorem lebesgue_outer_measure_empty {
    lebesgue_outer_measure(Set[Real].empty_set) = Real.0
} by {
    empty_cost_set_infimum_zero
    is_set_infimum(lebesgue_cost_set(Set[Real].empty_set), Real.0)
    exists(v: Real) { is_set_infimum(lebesgue_cost_set(Set[Real].empty_set), v) }
    lebesgue_cost_has_infimum(Set[Real].empty_set)
    lebesgue_outer_measure_is_infimum(Set[Real].empty_set)
    is_set_infimum(lebesgue_cost_set(Set[Real].empty_set), lebesgue_outer_measure(Set[Real].empty_set))
    set_infimum_unique(lebesgue_cost_set(Set[Real].empty_set), lebesgue_outer_measure(Set[Real].empty_set), Real.0)
    lebesgue_outer_measure(Set[Real].empty_set) = Real.0
}

/// The lower endpoint sequence of the self-cover of an interval.
define self_cover_lower(a: Real, b: Real, n: Nat) -> Real {
    if n = Nat.0 { a } else { Real.0 }
}

/// The upper endpoint sequence of the self-cover of an interval.
define self_cover_upper(a: Real, b: Real, n: Nat) -> Real {
    if n = Nat.0 { b } else { Real.0 }
}

/// The interval itself is the zero-th member of its self-cover family.
theorem self_cover_interval_at_zero(a: Real, b: Real) {
    interval_family(self_cover_lower(a, b), self_cover_upper(a, b))(Nat.0) = closed_interval_set(a, b)
} by {
    interval_family(self_cover_lower(a, b), self_cover_upper(a, b))(Nat.0) =
        closed_interval_set(self_cover_lower(a, b, Nat.0), self_cover_upper(a, b, Nat.0))
    self_cover_lower(a, b, Nat.0) = a
    self_cover_upper(a, b, Nat.0) = b
    closed_interval_set(self_cover_lower(a, b, Nat.0), self_cover_upper(a, b, Nat.0)) = closed_interval_set(a, b)
    interval_family(self_cover_lower(a, b), self_cover_upper(a, b))(Nat.0) = closed_interval_set(a, b)
}

/// The self-cover of a proper interval is proper.
theorem self_cover_is_proper(a: Real, b: Real) {
    a <= b implies forall(n: Nat) {
        is_proper_interval(self_cover_lower(a, b, n), self_cover_upper(a, b, n))
    }
} by {
    if a <= b {
        forall(n: Nat) {
            if n = Nat.0 {
                self_cover_lower(a, b, n) = a
                self_cover_upper(a, b, n) = b
                is_proper_interval(self_cover_lower(a, b, n), self_cover_upper(a, b, n)) =
                    (self_cover_lower(a, b, n) <= self_cover_upper(a, b, n))
                is_proper_interval(self_cover_lower(a, b, n), self_cover_upper(a, b, n))
            }
            if n != Nat.0 {
                self_cover_lower(a, b, n) = Real.0
                self_cover_upper(a, b, n) = Real.0
                lte_refl(Real.0)
                Real.0 <= Real.0
                is_proper_interval(self_cover_lower(a, b, n), self_cover_upper(a, b, n)) =
                    (self_cover_lower(a, b, n) <= self_cover_upper(a, b, n))
                is_proper_interval(self_cover_lower(a, b, n), self_cover_upper(a, b, n))
            }
            is_proper_interval(self_cover_lower(a, b, n), self_cover_upper(a, b, n))
        }
    }
}

/// The self-cover of an interval covers the interval.
theorem self_cover_covers_interval(a: Real, b: Real) {
    closed_interval_set(a, b).subset(interval_family_union(self_cover_lower(a, b), self_cover_upper(a, b)))
} by {
    forall(x: Real) {
        if closed_interval_set(a, b).contains(x) {
            self_cover_interval_at_zero(a, b)
            interval_family(self_cover_lower(a, b), self_cover_upper(a, b))(Nat.0) = closed_interval_set(a, b)
            interval_family(self_cover_lower(a, b), self_cover_upper(a, b))(Nat.0).contains(x)
            indexed_union_contains_of_contains(interval_family(self_cover_lower(a, b), self_cover_upper(a, b)), Nat.0, x)
            indexed_union(interval_family(self_cover_lower(a, b), self_cover_upper(a, b))).contains(x)
            interval_family_union(self_cover_lower(a, b), self_cover_upper(a, b)).contains(x)
        }
    }
}

/// The length sequence of the self-cover is the delta sequence at the length.
theorem self_cover_lengths_delta(a: Real, b: Real) {
    forall(k: Nat) {
        interval_length(self_cover_lower(a, b, k), self_cover_upper(a, b, k)) = delta_seq(b - a, k)
    }
} by {
    forall(k: Nat) {
        if k = Nat.0 {
            self_cover_lower(a, b, k) = a
            self_cover_upper(a, b, k) = b
            interval_length(self_cover_lower(a, b, k), self_cover_upper(a, b, k)) = b - a
            delta_seq(b - a, k) = b - a
            interval_length(self_cover_lower(a, b, k), self_cover_upper(a, b, k)) = delta_seq(b - a, k)
        }
        if k != Nat.0 {
            self_cover_lower(a, b, k) = Real.0
            self_cover_upper(a, b, k) = Real.0
            interval_length(self_cover_lower(a, b, k), self_cover_upper(a, b, k)) = Real.0 - Real.0
            Real.0 - Real.0 = Real.0
            interval_length(self_cover_lower(a, b, k), self_cover_upper(a, b, k)) = Real.0
            delta_seq(b - a, k) = Real.0
            interval_length(self_cover_lower(a, b, k), self_cover_upper(a, b, k)) = delta_seq(b - a, k)
        }
        interval_length(self_cover_lower(a, b, k), self_cover_upper(a, b, k)) = delta_seq(b - a, k)
    }
}

/// The cost sequence of the self-cover converges to the length.
theorem self_cover_cost_converges(a: Real, b: Real) {
    converges_to(interval_cost_seq(self_cover_lower(a, b), self_cover_upper(a, b)), b - a)
} by {
    forall(n: Nat) {
        interval_cost_seq(self_cover_lower(a, b), self_cover_upper(a, b), n) =
            partial(function(k: Nat) { interval_length(self_cover_lower(a, b, k), self_cover_upper(a, b, k)) }, n)
        self_cover_lengths_delta(a, b)
        partial(function(k: Nat) { interval_length(self_cover_lower(a, b, k), self_cover_upper(a, b, k)) }, n) =
            partial(delta_seq(b - a), n)
        interval_cost_seq(self_cover_lower(a, b), self_cover_upper(a, b), n) = partial(delta_seq(b - a), n)
    }
    forall(i: Nat) {
        if Nat.1 <= i {
            partial_delta_seq_later(b - a, i)
            partial(delta_seq(b - a), i) = b - a
        }
    }
    eventual_eq(function(m: Nat) { partial(delta_seq(b - a), m) }, b - a)
    eventual_eq_imp_converges_to(function(m: Nat) { partial(delta_seq(b - a), m) }, b - a)
    converges_to(function(m: Nat) { partial(delta_seq(b - a), m) }, b - a)
    converges_to_of_pointwise_eq(
        function(m: Nat) { partial(delta_seq(b - a), m) },
        interval_cost_seq(self_cover_lower(a, b), self_cover_upper(a, b)), b - a)
    converges_to(interval_cost_seq(self_cover_lower(a, b), self_cover_upper(a, b)), b - a)
}

/// The self-cover witnesses that the length is a cost of the interval.
theorem self_cover_is_interval_cover_cost(a: Real, b: Real) {
    a <= b implies lebesgue_cost_set(closed_interval_set(a, b)).contains(b - a)
} by {
    if a <= b {
        self_cover_is_proper(a, b)
        self_cover_covers_interval(a, b)
        self_cover_cost_converges(a, b)
        exists(x: Nat -> Real, y: Nat -> Real) {
            (forall(n: Nat) { is_proper_interval(x(n), y(n)) })
                and closed_interval_set(a, b).subset(interval_family_union(x, y))
                and converges_to(interval_cost_seq(x, y), b - a)
        }
        is_interval_cover_cost(closed_interval_set(a, b), b - a)
        lebesgue_cost_set_contains_eq(closed_interval_set(a, b), b - a)
        lebesgue_cost_set(closed_interval_set(a, b)).contains(b - a)
    }
}

/// The cost set of a proper closed interval has an infimum.
theorem interval_cost_has_infimum(a: Real, b: Real) {
    a <= b implies lebesgue_cost_has_infimum(closed_interval_set(a, b))
} by {
    if a <= b {
        self_cover_is_interval_cover_cost(a, b)
        lebesgue_cost_set(closed_interval_set(a, b)).contains(b - a)
        is_nonempty(lebesgue_cost_set(closed_interval_set(a, b)))
        forall(c: Real) {
            if lebesgue_cost_set(closed_interval_set(a, b)).contains(c) {
                lebesgue_cost_nonneg(closed_interval_set(a, b), c)
                Real.0 <= c
            }
        }
        is_set_lower_bound(lebesgue_cost_set(closed_interval_set(a, b)), Real.0)
        exists(lb: Real) { is_set_lower_bound(lebesgue_cost_set(closed_interval_set(a, b)), lb) }
        infimum_exists_of_nonempty_bounded_below(lebesgue_cost_set(closed_interval_set(a, b)))
        exists(v: Real) { is_set_infimum(lebesgue_cost_set(closed_interval_set(a, b)), v) }
        lebesgue_cost_has_infimum(closed_interval_set(a, b))
    }
}

/// The Lebesgue outer measure of a closed interval is at most its length.
theorem lebesgue_outer_measure_interval_upper(a: Real, b: Real) {
    a <= b implies lebesgue_outer_measure(closed_interval_set(a, b)) <= b - a
} by {
    if a <= b {
        interval_cost_has_infimum(a, b)
        lebesgue_cost_has_infimum(closed_interval_set(a, b))
        lebesgue_outer_measure_is_infimum(closed_interval_set(a, b))
        is_set_infimum(lebesgue_cost_set(closed_interval_set(a, b)), lebesgue_outer_measure(closed_interval_set(a, b)))
        self_cover_is_interval_cover_cost(a, b)
        lebesgue_cost_set(closed_interval_set(a, b)).contains(b - a)
        is_set_infimum(lebesgue_cost_set(closed_interval_set(a, b)), lebesgue_outer_measure(closed_interval_set(a, b))) =
            (is_set_lower_bound(lebesgue_cost_set(closed_interval_set(a, b)), lebesgue_outer_measure(closed_interval_set(a, b)))
                and forall(x: Real) {
                    is_set_lower_bound(lebesgue_cost_set(closed_interval_set(a, b)), x) implies x <= lebesgue_outer_measure(closed_interval_set(a, b))
                })
        is_set_lower_bound(lebesgue_cost_set(closed_interval_set(a, b)), lebesgue_outer_measure(closed_interval_set(a, b)))
        is_set_lower_bound(lebesgue_cost_set(closed_interval_set(a, b)), lebesgue_outer_measure(closed_interval_set(a, b))) =
            forall(x: Real) {
                lebesgue_cost_set(closed_interval_set(a, b)).contains(x) implies lebesgue_outer_measure(closed_interval_set(a, b)) <= x
            }
        lebesgue_cost_set(closed_interval_set(a, b)).contains(b - a) implies lebesgue_outer_measure(closed_interval_set(a, b)) <= b - a
        lebesgue_outer_measure(closed_interval_set(a, b)) <= b - a
    }
}

/// The Lebesgue outer measure of a degenerate interval is zero.
theorem lebesgue_outer_measure_point(a: Real) {
    lebesgue_outer_measure(closed_interval_set(a, a)) = Real.0
} by {
    lte_refl(a)
    a <= a
    interval_cost_has_infimum(a, a)
    lebesgue_cost_has_infimum(closed_interval_set(a, a))
    lebesgue_outer_measure_is_infimum(closed_interval_set(a, a))
    is_set_infimum(lebesgue_cost_set(closed_interval_set(a, a)), lebesgue_outer_measure(closed_interval_set(a, a)))
    self_cover_is_interval_cover_cost(a, a)
    lebesgue_cost_set(closed_interval_set(a, a)).contains(a - a)
    a - a = Real.0
    lebesgue_cost_set(closed_interval_set(a, a)).contains(Real.0)
    is_set_infimum(lebesgue_cost_set(closed_interval_set(a, a)), lebesgue_outer_measure(closed_interval_set(a, a))) =
        (is_set_lower_bound(lebesgue_cost_set(closed_interval_set(a, a)), lebesgue_outer_measure(closed_interval_set(a, a)))
            and forall(x: Real) {
                is_set_lower_bound(lebesgue_cost_set(closed_interval_set(a, a)), x) implies x <= lebesgue_outer_measure(closed_interval_set(a, a))
            })
    is_set_lower_bound(lebesgue_cost_set(closed_interval_set(a, a)), lebesgue_outer_measure(closed_interval_set(a, a)))
    is_set_lower_bound(lebesgue_cost_set(closed_interval_set(a, a)), lebesgue_outer_measure(closed_interval_set(a, a))) =
        forall(x: Real) {
            lebesgue_cost_set(closed_interval_set(a, a)).contains(x) implies lebesgue_outer_measure(closed_interval_set(a, a)) <= x
        }
    lebesgue_cost_set(closed_interval_set(a, a)).contains(Real.0) implies lebesgue_outer_measure(closed_interval_set(a, a)) <= Real.0
    lebesgue_outer_measure(closed_interval_set(a, a)) <= Real.0
    forall(c: Real) {
        if lebesgue_cost_set(closed_interval_set(a, a)).contains(c) {
            lebesgue_cost_nonneg(closed_interval_set(a, a), c)
            Real.0 <= c
        }
    }
    is_set_lower_bound(lebesgue_cost_set(closed_interval_set(a, a)), Real.0)
    is_set_infimum(lebesgue_cost_set(closed_interval_set(a, a)), lebesgue_outer_measure(closed_interval_set(a, a))) =
        (is_set_lower_bound(lebesgue_cost_set(closed_interval_set(a, a)), lebesgue_outer_measure(closed_interval_set(a, a)))
            and forall(x: Real) {
                is_set_lower_bound(lebesgue_cost_set(closed_interval_set(a, a)), x) implies x <= lebesgue_outer_measure(closed_interval_set(a, a))
            })
    forall(x: Real) {
        is_set_lower_bound(lebesgue_cost_set(closed_interval_set(a, a)), x) implies x <= lebesgue_outer_measure(closed_interval_set(a, a))
    }
    is_set_lower_bound(lebesgue_cost_set(closed_interval_set(a, a)), Real.0) implies Real.0 <= lebesgue_outer_measure(closed_interval_set(a, a))
    Real.0 <= lebesgue_outer_measure(closed_interval_set(a, a))
    lte_antisymm(lebesgue_outer_measure(closed_interval_set(a, a)), Real.0)
    lebesgue_outer_measure(closed_interval_set(a, a)) = Real.0
}

/// Cost sets are antitone in the covered set: a subset of `A` has no fewer
/// costs than `A`.
theorem lebesgue_cost_set_antitone(s: Set[Real], t: Set[Real]) {
    s.subset(t) implies lebesgue_cost_set(t).subset(lebesgue_cost_set(s))
} by {
    if s.subset(t) {
        forall(c: Real) {
            if lebesgue_cost_set(t).contains(c) {
                lebesgue_cost_set_contains_eq(t, c)
                is_interval_cover_cost(t, c)
                let (a: Nat -> Real, b: Nat -> Real) satisfy {
                    (forall(n: Nat) { is_proper_interval(a(n), b(n)) })
                        and t.subset(interval_family_union(a, b))
                        and converges_to(interval_cost_seq(a, b), c)
                }
                subset_trans(s, t, interval_family_union(a, b))
                s.subset(t) and t.subset(interval_family_union(a, b))
                s.subset(interval_family_union(a, b))
                is_interval_cover_cost(s, c)
                lebesgue_cost_set_contains_eq(s, c)
                lebesgue_cost_set(s).contains(c)
            }
        }
    }
}

/// The Lebesgue outer measure is monotone: a subset has no larger measure.
theorem lebesgue_outer_measure_monotone(s: Set[Real], t: Set[Real]) {
    s.subset(t) and lebesgue_cost_has_infimum(s) and lebesgue_cost_has_infimum(t)
        implies lebesgue_outer_measure(s) <= lebesgue_outer_measure(t)
} by {
    if s.subset(t) and lebesgue_cost_has_infimum(s) and lebesgue_cost_has_infimum(t) {
        lebesgue_outer_measure_is_infimum(s)
        lebesgue_outer_measure_is_infimum(t)
        is_set_infimum(lebesgue_cost_set(s), lebesgue_outer_measure(s))
        is_set_infimum(lebesgue_cost_set(t), lebesgue_outer_measure(t))
        lebesgue_cost_set_antitone(s, t)
        lebesgue_cost_set(t).subset(lebesgue_cost_set(s))
        forall(c: Real) {
            if lebesgue_cost_set(t).contains(c) {
                lebesgue_cost_set(t).subset(lebesgue_cost_set(s)) = forall(x: Real) {
                    lebesgue_cost_set(t).contains(x) implies lebesgue_cost_set(s).contains(x)
                }
                lebesgue_cost_set(s).contains(c)
                is_set_infimum(lebesgue_cost_set(s), lebesgue_outer_measure(s)) =
                    (is_set_lower_bound(lebesgue_cost_set(s), lebesgue_outer_measure(s))
                        and forall(x: Real) {
                            is_set_lower_bound(lebesgue_cost_set(s), x) implies x <= lebesgue_outer_measure(s)
                        })
                is_set_lower_bound(lebesgue_cost_set(s), lebesgue_outer_measure(s))
                is_set_lower_bound(lebesgue_cost_set(s), lebesgue_outer_measure(s)) =
                    forall(x: Real) {
                        lebesgue_cost_set(s).contains(x) implies lebesgue_outer_measure(s) <= x
                    }
                lebesgue_cost_set(s).contains(c) implies lebesgue_outer_measure(s) <= c
                lebesgue_outer_measure(s) <= c
            }
        }
        is_set_lower_bound(lebesgue_cost_set(t), lebesgue_outer_measure(s))
        is_set_infimum(lebesgue_cost_set(t), lebesgue_outer_measure(t)) =
            (is_set_lower_bound(lebesgue_cost_set(t), lebesgue_outer_measure(t))
                and forall(x: Real) {
                    is_set_lower_bound(lebesgue_cost_set(t), x) implies x <= lebesgue_outer_measure(t)
                })
        forall(x: Real) {
            is_set_lower_bound(lebesgue_cost_set(t), x) implies x <= lebesgue_outer_measure(t)
        }
        is_set_lower_bound(lebesgue_cost_set(t), lebesgue_outer_measure(s)) implies lebesgue_outer_measure(s) <= lebesgue_outer_measure(t)
        lebesgue_outer_measure(s) <= lebesgue_outer_measure(t)
    }
}

/// The Lebesgue outer measure is nonnegative where it is the infimum of the
/// cost set.
theorem lebesgue_outer_measure_nonneg(s: Set[Real]) {
    lebesgue_cost_has_infimum(s) implies Real.0 <= lebesgue_outer_measure(s)
} by {
    if lebesgue_cost_has_infimum(s) {
        lebesgue_outer_measure_is_infimum(s)
        is_set_infimum(lebesgue_cost_set(s), lebesgue_outer_measure(s))
        forall(c: Real) {
            if lebesgue_cost_set(s).contains(c) {
                lebesgue_cost_nonneg(s, c)
                Real.0 <= c
            }
        }
        is_set_lower_bound(lebesgue_cost_set(s), Real.0)
        is_set_infimum(lebesgue_cost_set(s), lebesgue_outer_measure(s)) =
            (is_set_lower_bound(lebesgue_cost_set(s), lebesgue_outer_measure(s))
                and forall(x: Real) {
                    is_set_lower_bound(lebesgue_cost_set(s), x) implies x <= lebesgue_outer_measure(s)
                })
        forall(x: Real) {
            is_set_lower_bound(lebesgue_cost_set(s), x) implies x <= lebesgue_outer_measure(s)
        }
        is_set_lower_bound(lebesgue_cost_set(s), Real.0) implies Real.0 <= lebesgue_outer_measure(s)
        Real.0 <= lebesgue_outer_measure(s)
    }
}

/// A set is Lebesgue measurable when it splits every test set under the
/// Lebesgue outer measure (Carathéodory's criterion).
define lebesgue_measurable(a: Set[Real]) -> Bool {
    forall(e: Set[Real]) {
        lebesgue_outer_measure(e) =
            lebesgue_outer_measure(e.intersection(a)) + lebesgue_outer_measure(e.difference(a))
    }
}

/// The empty set is Lebesgue measurable.
theorem lebesgue_measurable_empty {
    lebesgue_measurable(Set[Real].empty_set)
} by {
    forall(e: Set[Real]) {
        intersection_with_empty_is_empty(e)
        e.intersection(Set[Real].empty_set) = Set[Real].empty_set
        difference_empty_is_self(e)
        e.difference(Set[Real].empty_set) = e
        lebesgue_outer_measure_empty
        lebesgue_outer_measure(Set[Real].empty_set) = Real.0
        lebesgue_outer_measure(e) =
            lebesgue_outer_measure(e.intersection(Set[Real].empty_set)) + lebesgue_outer_measure(e.difference(Set[Real].empty_set))
    }
}

/// The whole real line is Lebesgue measurable.
theorem lebesgue_measurable_universal {
    lebesgue_measurable(Set[Real].universal_set)
} by {
    forall(e: Set[Real]) {
        intersection_with_universal_is_self(e)
        e.intersection(Set[Real].universal_set) = e
        difference_universal_is_empty(e)
        e.difference(Set[Real].universal_set) = Set[Real].empty_set
        lebesgue_outer_measure_empty
        lebesgue_outer_measure(Set[Real].empty_set) = Real.0
        lebesgue_outer_measure(e) =
            lebesgue_outer_measure(e.intersection(Set[Real].universal_set)) + lebesgue_outer_measure(e.difference(Set[Real].universal_set))
    }
}

/// Lebesgue measurability is closed under complement.
theorem lebesgue_measurable_compl(a: Set[Real]) {
    lebesgue_measurable(a) implies lebesgue_measurable(a.c)
} by {
    if lebesgue_measurable(a) {
        forall(e: Set[Real]) {
            lebesgue_measurable(a) = forall(f: Set[Real]) {
                lebesgue_outer_measure(f) =
                    lebesgue_outer_measure(f.intersection(a)) + lebesgue_outer_measure(f.difference(a))
            }
            lebesgue_outer_measure(e) =
                lebesgue_outer_measure(e.intersection(a)) + lebesgue_outer_measure(e.difference(a))
            intersection_compl_is_difference(e, a)
            e.intersection(a.c) = e.difference(a)
            difference_compl_is_intersection(e, a)
            e.difference(a.c) = e.intersection(a)
            lebesgue_outer_measure(e) =
                lebesgue_outer_measure(e.intersection(a.c)) + lebesgue_outer_measure(e.difference(a.c))
        }
    }
}

/// Lebesgue measurability is closed under binary union.
theorem lebesgue_measurable_union(a: Set[Real], b: Set[Real]) {
    lebesgue_measurable(a) and lebesgue_measurable(b)
        implies lebesgue_measurable(a.union(b))
} by {
    if lebesgue_measurable(a) and lebesgue_measurable(b) {
        lebesgue_measurable(a) = forall(f: Set[Real]) {
            lebesgue_outer_measure(f) =
                lebesgue_outer_measure(f.intersection(a)) + lebesgue_outer_measure(f.difference(a))
        }
        lebesgue_measurable(b) = forall(g: Set[Real]) {
            lebesgue_outer_measure(g) =
                lebesgue_outer_measure(g.intersection(b)) + lebesgue_outer_measure(g.difference(b))
        }
        forall(e: Set[Real]) {
            lebesgue_outer_measure(e) =
                lebesgue_outer_measure(e.intersection(a)) + lebesgue_outer_measure(e.difference(a))
            lebesgue_outer_measure(e.intersection(a)) =
                lebesgue_outer_measure((e.intersection(a)).intersection(b)) + lebesgue_outer_measure((e.intersection(a)).difference(b))
            lebesgue_outer_measure(e.difference(a)) =
                lebesgue_outer_measure((e.difference(a)).intersection(b)) + lebesgue_outer_measure((e.difference(a)).difference(b))
            lebesgue_outer_measure(e) =
                lebesgue_outer_measure((e.intersection(a)).intersection(b)) + lebesgue_outer_measure((e.intersection(a)).difference(b)) + lebesgue_outer_measure((e.difference(a)).intersection(b)) + lebesgue_outer_measure((e.difference(a)).difference(b))
            difference_iterated_union(e, b, a)
            (e.difference(a)).difference(b) = e.difference(b.union(a))
            union_comm(b, a)
            b.union(a) = a.union(b)
            (e.difference(a)).difference(b) = e.difference(a.union(b))
            lebesgue_outer_measure(e) =
                lebesgue_outer_measure((e.intersection(a)).intersection(b)) + lebesgue_outer_measure((e.intersection(a)).difference(b)) + lebesgue_outer_measure((e.difference(a)).intersection(b)) + lebesgue_outer_measure(e.difference(a.union(b)))
            lebesgue_outer_measure(e.intersection(a.union(b))) =
                lebesgue_outer_measure((e.intersection(a.union(b))).intersection(a)) + lebesgue_outer_measure((e.intersection(a.union(b))).difference(a))
            intersection_union_intersect_left(e, a, b)
            (e.intersection(a.union(b))).intersection(a) = e.intersection(a)
            intersection_union_difference_left(e, a, b)
            (e.intersection(a.union(b))).difference(a) = (e.difference(a)).intersection(b)
            lebesgue_outer_measure(e.intersection(a.union(b))) =
                lebesgue_outer_measure(e.intersection(a)) + lebesgue_outer_measure((e.difference(a)).intersection(b))
            lebesgue_outer_measure(e.intersection(a)) =
                lebesgue_outer_measure((e.intersection(a)).intersection(b)) + lebesgue_outer_measure((e.intersection(a)).difference(b))
            lebesgue_outer_measure(e.intersection(a.union(b))) =
                lebesgue_outer_measure((e.intersection(a)).intersection(b)) + lebesgue_outer_measure((e.intersection(a)).difference(b)) + lebesgue_outer_measure((e.difference(a)).intersection(b))
            lebesgue_outer_measure((e.intersection(a)).intersection(b)) + lebesgue_outer_measure((e.intersection(a)).difference(b)) + lebesgue_outer_measure((e.difference(a)).intersection(b)) + lebesgue_outer_measure(e.difference(a.union(b))) =
                lebesgue_outer_measure(e.intersection(a.union(b))) + lebesgue_outer_measure(e.difference(a.union(b)))
            lebesgue_outer_measure(e) =
                lebesgue_outer_measure(e.intersection(a.union(b))) + lebesgue_outer_measure(e.difference(a.union(b)))
        }
    }
}

/// Lebesgue measurability is closed under binary intersection.
theorem lebesgue_measurable_intersection(a: Set[Real], b: Set[Real]) {
    lebesgue_measurable(a) and lebesgue_measurable(b)
        implies lebesgue_measurable(a.intersection(b))
} by {
    if lebesgue_measurable(a) and lebesgue_measurable(b) {
        lebesgue_measurable_compl(a)
        lebesgue_measurable_compl(b)
        lebesgue_measurable_union(a.c, b.c)
        lebesgue_measurable(a.c.union(b.c))
        union_compl_is_intersection_compl(a, b)
        (a.c.union(b.c)).c = a.intersection(b)
        lebesgue_measurable_compl(a.c.union(b.c))
        lebesgue_measurable(a.intersection(b))
    }
}

/// Lebesgue measurability is closed under set difference.
theorem lebesgue_measurable_difference(a: Set[Real], b: Set[Real]) {
    lebesgue_measurable(a) and lebesgue_measurable(b)
        implies lebesgue_measurable(a.difference(b))
} by {
    if lebesgue_measurable(a) and lebesgue_measurable(b) {
        lebesgue_measurable_compl(b)
        lebesgue_measurable(b.c)
        lebesgue_measurable_intersection(a, b.c)
        lebesgue_measurable(a.intersection(b.c))
        difference_eq_intersection_compl(a, b)
        a.difference(b) = a.intersection(b.c)
        lebesgue_measurable(a.difference(b))
    }
}

/// The Lebesgue measurable predicate satisfies the empty-set axiom of a
/// sigma-algebra.
theorem lebesgue_measurable_has_empty {
    has_measurable_empty_constraint(lebesgue_measurable)
} by {
    lebesgue_measurable_empty
}

/// The Lebesgue measurable predicate is closed under complement.
theorem lebesgue_measurable_closed_compl {
    closed_under_compl_constraint(lebesgue_measurable)
} by {
    forall(s: Set[Real]) {
        lebesgue_measurable_compl(s)
    }
}

// TODO (Carathéodory's theorem): the collection of Lebesgue measurable sets
// is a sigma-algebra.  The finite closure results above (empty, universal,
// complement, binary union, binary intersection, difference) are proved, and
// the missing piece is closure under countable union:
//     closed_under_countable_union_constraint(lebesgue_measurable)
// The classical proof disjointifies a countable family (B_n = A_n \ ⋃_{i<n}
// A_i), proves that the outer measure splits each test set over the finite
// unions by induction, and passes to the limit using countable subadditivity
// and monotone convergence of the nonnegative partial sums.  In the current
// Real-valued setting the countable sums are represented by partial sums and
// the limit passage requires the partial sums to converge; the full proof is
// deferred (compare the TODO in measure.ac).

// TODO (normalization, lower bound): the full equality
//     lebesgue_outer_measure(closed_interval_set(a, b)) = b - a   (a <= b)
// also needs the reverse inequality: every countable interval cover of
// [a, b] has total cost at least b - a.  The classical proof first reduces
// to a finite subcover via Heine-Borel compactness of [a, b] in the
// open-cover form (the library's real compactness is closed-and-bounded,
// and the open-cover version for intervals is not yet formalized), then
// bounds a finite cover by the sum of its lengths, and finally passes to
// the limit.  The upper bound `lebesgue_outer_measure_interval_upper` and
// the degenerate case `lebesgue_outer_measure_point` above are proved.

// TODO (countable subadditivity): for sets `a`, `b` whose cost sets have
// infima, the binary subadditivity
//     lebesgue_outer_measure(a.union(b)) <= lebesgue_outer_measure(a) + lebesgue_outer_measure(b)
// holds: interleaving a cover of `a` with a cover of `b` produces a cover of
// `a.union(b)` whose partial sums converge to the sum of the two costs, and
// the infima compare accordingly.  The interleaving of the endpoint
// sequences and the convergence of the interleaved partial sums are the
// deferred steps.
