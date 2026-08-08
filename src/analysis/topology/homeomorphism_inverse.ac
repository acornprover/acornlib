from analysis.topology.topological_space import TopologicalSpace, is_continuous, is_homeomorphism,
    is_homeomorphism_iff, is_homeomorphism_continuous_fwd,
    is_homeomorphism_continuous_inv, is_homeomorphism_left_inv,
    is_homeomorphism_right_inv

/// Swapping the two maps of a homeomorphism pair yields a homeomorphism pair in
/// the opposite direction.
theorem homeomorphism_inverse[X: TopologicalSpace, Y: TopologicalSpace](
    f: X -> Y, g: Y -> X) {
    is_homeomorphism[X, Y](f, g) implies is_homeomorphism[Y, X](g, f)
} by {
    if is_homeomorphism[X, Y](f, g) {
        is_homeomorphism_continuous_fwd[X, Y](f, g)
        is_homeomorphism_continuous_inv[X, Y](f, g)
        forall(y: Y) {
            is_homeomorphism_right_inv[X, Y](f, g, y)
            f(g(y)) = y
        }
        forall(x: X) {
            is_homeomorphism_left_inv[X, Y](f, g, x)
            g(f(x)) = x
        }
        is_homeomorphism_iff[Y, X](g, f)
        is_homeomorphism[Y, X](g, f)
    }
}
