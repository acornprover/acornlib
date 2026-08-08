from data.basic.functions import compose, identity_fn
from nat import Nat, alt_suc_ne_zero
from data.basic.set import Set, indexed_union, indexed_intersection,
    indexed_union_contains_eq, indexed_union_contains_of_contains,
    indexed_union_contains_witness,
    indexed_intersection_contains_eq, indexed_intersection_contains_at,
    indexed_intersection_contains_of_forall,
    indexed_intersection_complement_eq_union_complement,
    indexed_union_complement_eq_intersection_complement,
    indexed_union_empty_of_forall_empty,
    set_preimage_indexed_union, set_preimage_indexed_intersection,
    empty_set_contains_eq, universal_set_contains_eq,
    empty_set_compl_is_universal, universal_set_compl_is_empty,
    set_eq_universal_of_forall_contains,
    union_contains_eq, union_contains_left, union_contains_right,
    union_contains_cases,
    intersection_contains_eq, difference_contains_eq,
    compl_contains_eq, set_ext, compl_of_compl_is_self,
    set_preimage, set_preimage_contains_eq, set_preimage_compose,
    set_preimage_empty, set_preimage_compl, subset_contains
from data.basic.witness import choose_or_default, choose_or_default_spec

/// True if the empty set is measurable under `is_meas`.
define has_measurable_empty_constraint[K](is_meas: Set[K] -> Bool) -> Bool {
    is_meas(Set[K].empty_set)
}

/// True if the measurable predicate `is_meas` is closed under complement.
define closed_under_compl_constraint[K](is_meas: Set[K] -> Bool) -> Bool {
    forall(s: Set[K]) {
        is_meas(s) implies is_meas(s.c)
    }
}

/// True if the measurable predicate `is_meas` is closed under countable union.
define closed_under_countable_union_constraint[K](is_meas: Set[K] -> Bool) -> Bool {
    forall(family: Nat -> Set[K]) {
        (forall(n: Nat) { is_meas(family(n)) })
            implies is_meas(indexed_union(family))
    }
}

/// A sigma-algebra on `K` is a collection of subsets of `K`
/// closed under complement and countable union, containing the empty set.
structure SigmaAlgebra[K] {
    /// True if the set is measurable in this sigma-algebra.
    is_measurable: Set[K] -> Bool
} constraint {
    has_measurable_empty_constraint(is_measurable)
    and closed_under_compl_constraint(is_measurable)
    and closed_under_countable_union_constraint(is_measurable)
}

/// The empty set is measurable in any sigma-algebra.
theorem sigma_algebra_measurable_empty[K](m: SigmaAlgebra[K]) {
    m.is_measurable(Set[K].empty_set)
} by {
}

/// Sigma-algebras are closed under complement.
theorem sigma_algebra_measurable_compl[K](m: SigmaAlgebra[K], s: Set[K]) {
    m.is_measurable(s) implies m.is_measurable(s.c)
} by {
    closed_under_compl_constraint(m.is_measurable) = forall(t: Set[K]) {
        m.is_measurable(t) implies m.is_measurable(t.c)
    }
}

/// Sigma-algebras are closed under countable union.
theorem sigma_algebra_measurable_indexed_union[K](m: SigmaAlgebra[K],
        family: Nat -> Set[K]) {
    (forall(n: Nat) { m.is_measurable(family(n)) })
        implies m.is_measurable(indexed_union(family))
} by {
    closed_under_countable_union_constraint(m.is_measurable)
    closed_under_countable_union_constraint(m.is_measurable) =
        forall(g: Nat -> Set[K]) {
            (forall(n: Nat) { m.is_measurable(g(n)) })
                implies m.is_measurable(indexed_union(g))
        }
}

/// The universal set is measurable in any sigma-algebra.
theorem sigma_algebra_measurable_universal[K](m: SigmaAlgebra[K]) {
    m.is_measurable(Set[K].universal_set)
} by {
    sigma_algebra_measurable_empty(m)
    sigma_algebra_measurable_compl(m, Set[K].empty_set)
    forall(x: K) {
        empty_set_contains_eq[K](x)
        compl_contains_eq(Set[K].empty_set, x)
        universal_set_contains_eq[K](x)
        (Set[K].empty_set).c.contains(x) = (Set[K].universal_set).contains(x)
    }
    set_ext((Set[K].empty_set).c, Set[K].universal_set)
}

/// A set is measurable iff its complement is measurable.
theorem sigma_algebra_measurable_compl_iff[K](m: SigmaAlgebra[K], s: Set[K]) {
    m.is_measurable(s.c) = m.is_measurable(s)
} by {
    if m.is_measurable(s.c) {
        sigma_algebra_measurable_compl(m, s.c)
        m.is_measurable(s.c.c)
        compl_of_compl_is_self(s)
        s.c.c = s
        m.is_measurable(s)
    }
    if m.is_measurable(s) {
        sigma_algebra_measurable_compl(m, s)
        m.is_measurable(s.c)
    }
}

/// Sigma-algebras are closed under countable intersection.
theorem sigma_algebra_measurable_indexed_intersection[K](m: SigmaAlgebra[K],
        family: Nat -> Set[K]) {
    (forall(n: Nat) { m.is_measurable(family(n)) })
        implies m.is_measurable(indexed_intersection(family))
} by {
    if forall(n: Nat) { m.is_measurable(family(n)) } {
        forall(n: Nat) {
            sigma_algebra_measurable_compl(m, family(n))
            m.is_measurable(family(n).c)
        }
        sigma_algebra_measurable_indexed_union(m,
            function(n: Nat) { family(n).c })
        m.is_measurable(indexed_union(function(n: Nat) { family(n).c }))
        indexed_union_complement_eq_intersection_complement[K, Nat](family)
        let lhs: Set[K] = indexed_union(function(n: Nat) { family(n).c })
        let rhs: Set[K] = indexed_intersection(family).c
        sigma_algebra_measurable_compl(m, indexed_intersection(family).c)
        m.is_measurable(indexed_intersection(family).c.c)
        compl_of_compl_is_self(indexed_intersection(family))
        m.is_measurable(indexed_intersection(family))
    }
}

/// The discrete measurable predicate: every set is measurable.
define discrete_measurable[K](s: Set[K]) -> Bool {
    true
}

/// The discrete measurable predicate satisfies the sigma-algebra axioms.
theorem discrete_measurable_is_sigma_algebra[K] {
    has_measurable_empty_constraint(discrete_measurable[K])
        and closed_under_compl_constraint(discrete_measurable[K])
        and closed_under_countable_union_constraint(discrete_measurable[K])
} by {
    forall(s: Set[K]) {
        if discrete_measurable[K](s) {
            discrete_measurable[K](s.c)
        }
    }
    closed_under_compl_constraint(discrete_measurable[K])
    forall(family: Nat -> Set[K]) {
        if forall(n: Nat) { discrete_measurable[K](family(n)) } {
            discrete_measurable[K](indexed_union(family))
        }
    }
    closed_under_countable_union_constraint(discrete_measurable[K])
}

/// The discrete sigma-algebra on `K`, in which every set is measurable.
let discrete_sigma_algebra[K]: SigmaAlgebra[K] satisfy {
    SigmaAlgebra.new(discrete_measurable[K]) = Option.some(discrete_sigma_algebra)
}

attributes SigmaAlgebra[K] {
    /// The discrete sigma-algebra on `K`, in which every set is measurable.
    let discrete: SigmaAlgebra[K] = discrete_sigma_algebra[K]
}

/// Every set is measurable in the discrete sigma-algebra.
theorem discrete_sigma_algebra_is_measurable[K](s: Set[K]) {
    SigmaAlgebra[K].discrete.is_measurable(s)
} by {
    SigmaAlgebra[K].discrete.is_measurable(s) = discrete_measurable[K](s)
}

/// Sigma-algebras are closed under finite (binary) union.
theorem sigma_algebra_measurable_union[K](m: SigmaAlgebra[K],
        a: Set[K], b: Set[K]) {
    m.is_measurable(a) and m.is_measurable(b)
        implies m.is_measurable(a.union(b))
} by {
    if m.is_measurable(a) and m.is_measurable(b) {
        let family: Nat -> Set[K] = function(n: Nat) {
            if n = Nat.0 { a } else { b }
        }
        forall(n: Nat) {
            if n = Nat.0 {
                m.is_measurable(family(n))
            }
            if n != Nat.0 {
                family(n) = b
                m.is_measurable(family(n))
            }
            m.is_measurable(family(n))
        }
        sigma_algebra_measurable_indexed_union(m, family)
        m.is_measurable(indexed_union(family))
        forall(x: K) {
            indexed_union_contains_eq(family, x)
            union_contains_eq(a, b, x)
            if indexed_union(family).contains(x) {
                indexed_union_contains_witness(family, x)
                let i: Nat satisfy { family(i).contains(x) }
                if i = Nat.0 {
                    union_contains_left(a, b, x)
                    a.union(b).contains(x)
                }
                if i != Nat.0 {
                    family(i) = b
                    union_contains_right(a, b, x)
                    a.union(b).contains(x)
                }
                a.union(b).contains(x)
            }
            if a.union(b).contains(x) {
                union_contains_cases(a, b, x)
                if a.contains(x) {
                    indexed_union_contains_of_contains(family, Nat.0, x)
                    indexed_union(family).contains(x)
                }
                if b.contains(x) {
                    family(Nat.0.suc) = b
                    indexed_union_contains_of_contains(family, Nat.0.suc, x)
                    indexed_union(family).contains(x)
                }
                indexed_union(family).contains(x)
            }
            indexed_union(family).contains(x) = a.union(b).contains(x)
        }
        set_ext(indexed_union(family), a.union(b))
        m.is_measurable(a.union(b))
    }
}

/// Sigma-algebras are closed under finite (binary) intersection.
theorem sigma_algebra_measurable_intersection[K](m: SigmaAlgebra[K],
        a: Set[K], b: Set[K]) {
    m.is_measurable(a) and m.is_measurable(b)
        implies m.is_measurable(a.intersection(b))
} by {
    if m.is_measurable(a) and m.is_measurable(b) {
        sigma_algebra_measurable_compl(m, a)
        sigma_algebra_measurable_compl(m, b)
        sigma_algebra_measurable_union(m, a.c, b.c)
        sigma_algebra_measurable_compl(m, a.c.union(b.c))
        forall(x: K) {
            compl_contains_eq(a.c.union(b.c), x)
            union_contains_eq(a.c, b.c, x)
            compl_contains_eq(a, x)
            compl_contains_eq(b, x)
            intersection_contains_eq(a, b, x)
            if a.intersection(b).contains(x) {
                a.c.union(b.c).c.contains(x)
            }
            if a.c.union(b.c).c.contains(x) {
                a.intersection(b).contains(x)
            }
            a.c.union(b.c).c.contains(x) = a.intersection(b).contains(x)
        }
        set_ext(a.c.union(b.c).c, a.intersection(b))
        m.is_measurable(a.intersection(b))
    }
}

/// Sigma-algebras are closed under set difference.
theorem sigma_algebra_measurable_difference[K](m: SigmaAlgebra[K],
        a: Set[K], b: Set[K]) {
    m.is_measurable(a) and m.is_measurable(b)
        implies m.is_measurable(a.difference(b))
} by {
    if m.is_measurable(a) and m.is_measurable(b) {
        sigma_algebra_measurable_compl(m, b)
        sigma_algebra_measurable_intersection(m, a, b.c)
        forall(x: K) {
            intersection_contains_eq(a, b.c, x)
            compl_contains_eq(b, x)
            difference_contains_eq(a, b, x)
            if a.intersection(b.c).contains(x) {
                a.difference(b).contains(x)
            }
            if a.difference(b).contains(x) {
                a.intersection(b.c).contains(x)
            }
            a.intersection(b.c).contains(x) = a.difference(b).contains(x)
        }
        set_ext(a.intersection(b.c), a.difference(b))
        m.is_measurable(a.difference(b))
    }
}

/// The trivial measurable predicate: only the empty and universal sets
/// are measurable.
define trivial_measurable[K](s: Set[K]) -> Bool {
    s = Set[K].empty_set or s = Set[K].universal_set
}

/// The empty set is trivially measurable.
theorem trivial_measurable_empty[K] {
    trivial_measurable[K](Set[K].empty_set)
} by {
}

/// The universal set is trivially measurable.
theorem trivial_measurable_universal[K] {
    trivial_measurable[K](Set[K].universal_set)
} by {
}

/// The trivial measurable predicate is closed under complement.
theorem trivial_measurable_compl[K](s: Set[K]) {
    trivial_measurable[K](s) implies trivial_measurable[K](s.c)
} by {
    if trivial_measurable[K](s) {
        if s = Set[K].empty_set {
            empty_set_compl_is_universal[K]
            trivial_measurable_universal[K]
            trivial_measurable[K](s.c)
        }
        if s = Set[K].universal_set {
            universal_set_compl_is_empty[K]
            trivial_measurable_empty[K]
            trivial_measurable[K](s.c)
        }
        trivial_measurable[K](s.c)
    }
}

/// If a countable family of trivially measurable sets contains the universal
/// set at some index, then its union is the universal set.
theorem trivial_measurable_indexed_union_universal[K](family: Nat -> Set[K], n: Nat) {
    family(n) = Set[K].universal_set
        implies indexed_union(family) = Set[K].universal_set
} by {
    if family(n) = Set[K].universal_set {
        forall(x: K) {
            universal_set_contains_eq[K](x)
            indexed_union_contains_of_contains(family, n, x)
            indexed_union(family).contains(x)
        }
        set_eq_universal_of_forall_contains(indexed_union(family))
    }
}

/// The trivial measurable predicate is closed under countable union.
theorem trivial_measurable_indexed_union[K](family: Nat -> Set[K]) {
    (forall(n: Nat) { trivial_measurable[K](family(n)) })
        implies trivial_measurable[K](indexed_union(family))
} by {
    if forall(n: Nat) { trivial_measurable[K](family(n)) } {
        if exists(n: Nat) { family(n) = Set[K].universal_set } {
            let n: Nat satisfy { family(n) = Set[K].universal_set }
            trivial_measurable_indexed_union_universal(family, n)
            trivial_measurable_universal[K]
            trivial_measurable[K](indexed_union(family))
        } else {
            forall(n: Nat) {
                family(n) = Set[K].empty_set
            }
            indexed_union_empty_of_forall_empty(family)
            indexed_union(family) = Set[K].empty_set
            trivial_measurable_empty[K]
            trivial_measurable[K](indexed_union(family))
        }
        trivial_measurable[K](indexed_union(family))
    }
}

/// The trivial measurable predicate satisfies the sigma-algebra axioms.
theorem trivial_measurable_is_sigma_algebra[K] {
    has_measurable_empty_constraint(trivial_measurable[K])
        and closed_under_compl_constraint(trivial_measurable[K])
        and closed_under_countable_union_constraint(trivial_measurable[K])
} by {
    trivial_measurable_empty[K]
    forall(s: Set[K]) {
        trivial_measurable_compl(s)
    }
    closed_under_compl_constraint(trivial_measurable[K])
    forall(family: Nat -> Set[K]) {
        trivial_measurable_indexed_union(family)
    }
    closed_under_countable_union_constraint(trivial_measurable[K])
}

/// The trivial sigma-algebra on `K`, in which only the empty set and the
/// universal set are measurable.
let trivial_sigma_algebra[K]: SigmaAlgebra[K] satisfy {
    SigmaAlgebra.new(trivial_measurable[K]) = Option.some(trivial_sigma_algebra)
}

attributes SigmaAlgebra[K] {
    /// The trivial sigma-algebra on `K`, in which only the empty and universal
    /// sets are measurable.
    let trivial: SigmaAlgebra[K] = trivial_sigma_algebra[K]
}

/// The refinement relation on sigma-algebras: `m1` is coarser than `m2`
/// when every set measurable in `m1` is also measurable in `m2`.
define sigma_algebra_le[K](m1: SigmaAlgebra[K], m2: SigmaAlgebra[K]) -> Bool {
    forall(s: Set[K]) {
        m1.is_measurable(s) implies m2.is_measurable(s)
    }
}

/// Refinement of sigma-algebras is reflexive.
theorem sigma_algebra_le_refl[K](m: SigmaAlgebra[K]) {
    sigma_algebra_le(m, m)
} by {
    forall(s: Set[K]) {
        if m.is_measurable(s) {
            m.is_measurable(s)
        }
    }
}

/// Refinement of sigma-algebras is transitive.
theorem sigma_algebra_le_trans[K](m1: SigmaAlgebra[K], m2: SigmaAlgebra[K],
        m3: SigmaAlgebra[K]) {
    sigma_algebra_le(m1, m2) and sigma_algebra_le(m2, m3)
        implies sigma_algebra_le(m1, m3)
} by {
    if sigma_algebra_le(m1, m2) and sigma_algebra_le(m2, m3) {
        forall(s: Set[K]) {
            if m1.is_measurable(s) {
                m2.is_measurable(s)
                m3.is_measurable(s)
            }
        }
    }
}

/// The trivial sigma-algebra is the coarsest: it refines into every
/// sigma-algebra on `K`.
theorem trivial_sigma_algebra_le[K](m: SigmaAlgebra[K]) {
    sigma_algebra_le(SigmaAlgebra[K].trivial, m)
} by {
    forall(s: Set[K]) {
        if SigmaAlgebra[K].trivial.is_measurable(s) {
            SigmaAlgebra[K].trivial.is_measurable(s) = trivial_measurable[K](s)
            s = Set[K].empty_set or s = Set[K].universal_set
            if s = Set[K].empty_set {
                sigma_algebra_measurable_empty(m)
                m.is_measurable(s)
            }
            if s = Set[K].universal_set {
                sigma_algebra_measurable_universal(m)
                m.is_measurable(s)
            }
            m.is_measurable(s)
        }
    }
}

/// The discrete sigma-algebra is the finest: every sigma-algebra on `K`
/// refines into it.
theorem sigma_algebra_le_discrete[K](m: SigmaAlgebra[K]) {
    sigma_algebra_le(m, SigmaAlgebra[K].discrete)
} by {
    forall(s: Set[K]) {
        if m.is_measurable(s) {
            discrete_sigma_algebra_is_measurable[K](s)
            SigmaAlgebra[K].discrete.is_measurable(s)
        }
    }
}

/// Equality of sigma-algebras as the symmetric refinement relation.
define sigma_algebra_equiv[K](m1: SigmaAlgebra[K], m2: SigmaAlgebra[K]) -> Bool {
    sigma_algebra_le(m1, m2) and sigma_algebra_le(m2, m1)
}

/// Sigma-algebra equivalence is reflexive.
theorem sigma_algebra_equiv_refl[K](m: SigmaAlgebra[K]) {
    sigma_algebra_equiv(m, m)
} by {
    sigma_algebra_le_refl(m)
}

/// Sigma-algebra equivalence is symmetric.
theorem sigma_algebra_equiv_symm[K](m1: SigmaAlgebra[K], m2: SigmaAlgebra[K]) {
    sigma_algebra_equiv(m1, m2) implies sigma_algebra_equiv(m2, m1)
} by {
    if sigma_algebra_equiv(m1, m2) {
        sigma_algebra_le(m1, m2) and sigma_algebra_le(m2, m1)
        sigma_algebra_le(m2, m1) and sigma_algebra_le(m1, m2)
    }
}

/// Sigma-algebra equivalence is transitive.
theorem sigma_algebra_equiv_trans[K](m1: SigmaAlgebra[K], m2: SigmaAlgebra[K],
        m3: SigmaAlgebra[K]) {
    sigma_algebra_equiv(m1, m2) and sigma_algebra_equiv(m2, m3)
        implies sigma_algebra_equiv(m1, m3)
} by {
    if sigma_algebra_equiv(m1, m2) and sigma_algebra_equiv(m2, m3) {
        sigma_algebra_le(m1, m2) and sigma_algebra_le(m2, m1)
        sigma_algebra_le_trans(m1, m2, m3)
        sigma_algebra_le_trans(m3, m2, m1)
        sigma_algebra_le(m3, m1)
        sigma_algebra_le(m1, m3) and sigma_algebra_le(m3, m1)
    }
}

/// Two sigma-algebras are equivalent exactly when they share the same
/// measurable predicate.
theorem sigma_algebra_equiv_iff_same_measurable[K](m1: SigmaAlgebra[K],
        m2: SigmaAlgebra[K]) {
    sigma_algebra_equiv(m1, m2) iff
        forall(s: Set[K]) { m1.is_measurable(s) = m2.is_measurable(s) }
} by {
    if sigma_algebra_equiv(m1, m2) {
        sigma_algebra_le(m1, m2)
        sigma_algebra_le(m2, m1)
        forall(s: Set[K]) {
            if m1.is_measurable(s) {
                m1.is_measurable(s) implies m2.is_measurable(s)
                m2.is_measurable(s)
            }
            if m2.is_measurable(s) {
                m2.is_measurable(s) implies m1.is_measurable(s)
                m1.is_measurable(s)
            }
            m1.is_measurable(s) = m2.is_measurable(s)
        }
    }
    if forall(s: Set[K]) { m1.is_measurable(s) = m2.is_measurable(s) } {
        forall(s: Set[K]) {
            if m1.is_measurable(s) {
                m2.is_measurable(s)
            }
            if m2.is_measurable(s) {
                m1.is_measurable(s)
            }
        }
        sigma_algebra_le(m2, m1)
    }
}

/// The empty set is measurable in the trivial sigma-algebra.
theorem trivial_sigma_algebra_is_measurable_empty[K] {
    SigmaAlgebra[K].trivial.is_measurable(Set[K].empty_set)
} by {
    trivial_measurable_empty[K]
}

/// The universal set is measurable in the trivial sigma-algebra.
theorem trivial_sigma_algebra_is_measurable_universal[K] {
    SigmaAlgebra[K].trivial.is_measurable(Set[K].universal_set)
} by {
    trivial_measurable_universal[K]
}

/// True if `f` is measurable with respect to the sigma-algebras `mA` on
/// the domain and `mB` on the codomain: the preimage of every measurable
/// set is measurable.
define is_measurable_fn[A, B](f: A -> B, mA: SigmaAlgebra[A],
        mB: SigmaAlgebra[B]) -> Bool {
    forall(t: Set[B]) {
        mB.is_measurable(t) implies mA.is_measurable(set_preimage(f, t))
    }
}

/// The identity function is measurable from any sigma-algebra to itself.
theorem is_measurable_fn_identity[A](m: SigmaAlgebra[A]) {
    is_measurable_fn(identity_fn[A], m, m)
} by {
    forall(t: Set[A]) {
        if m.is_measurable(t) {
            forall(x: A) {
                set_preimage_contains_eq(identity_fn[A], t, x)
                set_preimage(identity_fn[A], t).contains(x) = t.contains(x)
            }
            set_ext(set_preimage(identity_fn[A], t), t)
            m.is_measurable(set_preimage(identity_fn[A], t))
        }
    }
}

/// The composition of two measurable functions is measurable.
theorem is_measurable_fn_compose[A, B, C](f: B -> C, g: A -> B,
        mA: SigmaAlgebra[A], mB: SigmaAlgebra[B], mC: SigmaAlgebra[C]) {
    is_measurable_fn(g, mA, mB) and is_measurable_fn(f, mB, mC)
        implies is_measurable_fn(compose(f, g), mA, mC)
} by {
    if is_measurable_fn(g, mA, mB) and is_measurable_fn(f, mB, mC) {
        is_measurable_fn(g, mA, mB) = forall(u: Set[B]) {
            mB.is_measurable(u) implies mA.is_measurable(set_preimage(g, u))
        }
        is_measurable_fn(f, mB, mC) = forall(t: Set[C]) {
            mC.is_measurable(t) implies mB.is_measurable(set_preimage(f, t))
        }
        forall(t: Set[C]) {
            if mC.is_measurable(t) {
                mA.is_measurable(set_preimage(g, set_preimage(f, t)))
                set_preimage_compose(f, g, t)
                mA.is_measurable(set_preimage(compose(f, g), t))
            }
        }
    }
}

/// A measurable function pulls back measurable sets to measurable sets.
theorem is_measurable_fn_preimage_measurable[A, B](f: A -> B,
        mA: SigmaAlgebra[A], mB: SigmaAlgebra[B], t: Set[B]) {
    is_measurable_fn(f, mA, mB) and mB.is_measurable(t)
        implies mA.is_measurable(set_preimage(f, t))
} by {
    if is_measurable_fn(f, mA, mB) and mB.is_measurable(t) {
        is_measurable_fn(f, mA, mB) = forall(u: Set[B]) {
            mB.is_measurable(u) implies mA.is_measurable(set_preimage(f, u))
        }
        mA.is_measurable(set_preimage(f, t))
    }
}

/// Preimages of measurable binary unions under a measurable function are measurable.
theorem is_measurable_fn_preimage_union[A, B](f: A -> B,
        mA: SigmaAlgebra[A], mB: SigmaAlgebra[B], s: Set[B], t: Set[B]) {
    is_measurable_fn(f, mA, mB) and mB.is_measurable(s) and mB.is_measurable(t)
        implies mA.is_measurable(set_preimage(f, s.union(t)))
} by {
    if is_measurable_fn(f, mA, mB) and mB.is_measurable(s) and mB.is_measurable(t) {
        sigma_algebra_measurable_union(mB, s, t)
        mB.is_measurable(s.union(t))
        is_measurable_fn_preimage_measurable(f, mA, mB, s.union(t))
    }
}

/// Preimages of measurable binary intersections under a measurable function are measurable.
theorem is_measurable_fn_preimage_intersection[A, B](f: A -> B,
        mA: SigmaAlgebra[A], mB: SigmaAlgebra[B], s: Set[B], t: Set[B]) {
    is_measurable_fn(f, mA, mB) and mB.is_measurable(s) and mB.is_measurable(t)
        implies mA.is_measurable(set_preimage(f, s.intersection(t)))
} by {
    if is_measurable_fn(f, mA, mB) and mB.is_measurable(s) and mB.is_measurable(t) {
        sigma_algebra_measurable_intersection(mB, s, t)
        mB.is_measurable(s.intersection(t))
        is_measurable_fn_preimage_measurable(f, mA, mB, s.intersection(t))
    }
}

/// Preimages of measurable set differences under a measurable function are measurable.
theorem is_measurable_fn_preimage_difference[A, B](f: A -> B,
        mA: SigmaAlgebra[A], mB: SigmaAlgebra[B], s: Set[B], t: Set[B]) {
    is_measurable_fn(f, mA, mB) and mB.is_measurable(s) and mB.is_measurable(t)
        implies mA.is_measurable(set_preimage(f, s.difference(t)))
} by {
    if is_measurable_fn(f, mA, mB) and mB.is_measurable(s) and mB.is_measurable(t) {
        sigma_algebra_measurable_difference(mB, s, t)
        mB.is_measurable(s.difference(t))
        is_measurable_fn_preimage_measurable(f, mA, mB, s.difference(t))
    }
}

/// Preimages of countable unions of measurable sets under a measurable function are measurable.
theorem is_measurable_fn_preimage_indexed_union[A, B](f: A -> B,
        mA: SigmaAlgebra[A], mB: SigmaAlgebra[B], family: Nat -> Set[B]) {
    is_measurable_fn(f, mA, mB) and (forall(n: Nat) { mB.is_measurable(family(n)) })
        implies mA.is_measurable(set_preimage(f, indexed_union(family)))
} by {
    if is_measurable_fn(f, mA, mB) and forall(n: Nat) { mB.is_measurable(family(n)) } {
        sigma_algebra_measurable_indexed_union(mB, family)
        mB.is_measurable(indexed_union(family))
        is_measurable_fn_preimage_measurable(f, mA, mB, indexed_union(family))
    }
}

/// Preimages of countable intersections of measurable sets under a measurable function are measurable.
theorem is_measurable_fn_preimage_indexed_intersection[A, B](f: A -> B,
        mA: SigmaAlgebra[A], mB: SigmaAlgebra[B], family: Nat -> Set[B]) {
    is_measurable_fn(f, mA, mB) and (forall(n: Nat) { mB.is_measurable(family(n)) })
        implies mA.is_measurable(set_preimage(f, indexed_intersection(family)))
} by {
    if is_measurable_fn(f, mA, mB) and forall(n: Nat) { mB.is_measurable(family(n)) } {
        sigma_algebra_measurable_indexed_intersection(mB, family)
        mB.is_measurable(indexed_intersection(family))
        is_measurable_fn_preimage_measurable(f, mA, mB, indexed_intersection(family))
    }
}

/// Every function into the trivial sigma-algebra is measurable.
theorem is_measurable_fn_into_trivial[A, B](f: A -> B, mA: SigmaAlgebra[A]) {
    is_measurable_fn(f, mA, SigmaAlgebra[B].trivial)
} by {
    forall(t: Set[B]) {
        if SigmaAlgebra[B].trivial.is_measurable(t) {
            SigmaAlgebra[B].trivial.is_measurable(t) = trivial_measurable[B](t)
            t = Set[B].empty_set or t = Set[B].universal_set
            if t = Set[B].empty_set {
                forall(x: A) {
                    set_preimage_contains_eq(f, t, x)
                    empty_set_contains_eq[B](f(x))
                    empty_set_contains_eq[A](x)
                    set_preimage(f, t).contains(x) = Set[A].empty_set.contains(x)
                }
                set_ext(set_preimage(f, t), Set[A].empty_set)
                sigma_algebra_measurable_empty(mA)
                mA.is_measurable(set_preimage(f, t))
            }
            if t = Set[B].universal_set {
                forall(x: A) {
                    set_preimage_contains_eq(f, t, x)
                    universal_set_contains_eq[B](f(x))
                    universal_set_contains_eq[A](x)
                    set_preimage(f, t).contains(x) =
                        Set[A].universal_set.contains(x)
                }
                set_ext(set_preimage(f, t), Set[A].universal_set)
                sigma_algebra_measurable_universal(mA)
                mA.is_measurable(set_preimage(f, t))
            }
            mA.is_measurable(set_preimage(f, t))
        }
    }
}

/// Every function out of the discrete sigma-algebra is measurable.
theorem is_measurable_fn_from_discrete[A, B](f: A -> B, mB: SigmaAlgebra[B]) {
    is_measurable_fn(f, SigmaAlgebra[A].discrete, mB)
} by {
    forall(t: Set[B]) {
        if mB.is_measurable(t) {
            discrete_sigma_algebra_is_measurable[A](set_preimage(f, t))
            SigmaAlgebra[A].discrete.is_measurable(set_preimage(f, t))
        }
    }
}

/// If `f` is measurable into `mB` and `mB` refines into `mC` on the same
/// codomain, then `f` is still measurable when the codomain sigma-algebra
/// is replaced by the coarser one.
theorem is_measurable_fn_mono_codomain[A, B](f: A -> B,
        mA: SigmaAlgebra[A], mB: SigmaAlgebra[B], mC: SigmaAlgebra[B]) {
    is_measurable_fn(f, mA, mB) and sigma_algebra_le(mC, mB)
        implies is_measurable_fn(f, mA, mC)
} by {
    if is_measurable_fn(f, mA, mB) and sigma_algebra_le(mC, mB) {
        is_measurable_fn(f, mA, mB) = forall(u: Set[B]) {
            mB.is_measurable(u) implies mA.is_measurable(set_preimage(f, u))
        }
        sigma_algebra_le(mC, mB) = forall(s: Set[B]) {
            mC.is_measurable(s) implies mB.is_measurable(s)
        }
        forall(t: Set[B]) {
            if mC.is_measurable(t) {
                mA.is_measurable(set_preimage(f, t))
            }
        }
    }
}

/// Two sigma-algebras with the same measurable predicate are equal.
theorem sigma_algebra_eq_of_same_measurable[K](m1: SigmaAlgebra[K],
        m2: SigmaAlgebra[K]) {
    (forall(s: Set[K]) { m1.is_measurable(s) = m2.is_measurable(s) })
        implies m1 = m2
} by {
    if forall(s: Set[K]) { m1.is_measurable(s) = m2.is_measurable(s) } {
        m1.is_measurable = function(s: Set[K]) { m1.is_measurable(s) }
        m2.is_measurable = function(s: Set[K]) { m2.is_measurable(s) }
        m1.is_measurable = m2.is_measurable
    }
}

/// Equivalent sigma-algebras are equal.
theorem sigma_algebra_eq_of_equiv[K](m1: SigmaAlgebra[K], m2: SigmaAlgebra[K]) {
    sigma_algebra_equiv(m1, m2) implies m1 = m2
} by {
    if sigma_algebra_equiv(m1, m2) {
        sigma_algebra_equiv_iff_same_measurable(m1, m2)
        forall(s: Set[K]) { m1.is_measurable(s) = m2.is_measurable(s) }
        sigma_algebra_eq_of_same_measurable(m1, m2)
    }
}

/// Refinement of sigma-algebras is antisymmetric.
theorem sigma_algebra_le_antisymm[K](m1: SigmaAlgebra[K], m2: SigmaAlgebra[K]) {
    sigma_algebra_le(m1, m2) and sigma_algebra_le(m2, m1)
        implies m1 = m2
} by {
    if sigma_algebra_le(m1, m2) and sigma_algebra_le(m2, m1) {
        sigma_algebra_equiv(m1, m2)
        sigma_algebra_eq_of_equiv(m1, m2)
    }
}

/// If `f` is measurable and `g` agrees with `f` pointwise, then `g` is measurable.
theorem is_measurable_fn_pointwise_eq[A, B](f: A -> B, g: A -> B,
        mA: SigmaAlgebra[A], mB: SigmaAlgebra[B]) {
    is_measurable_fn(f, mA, mB) and (forall(x: A) { f(x) = g(x) })
        implies is_measurable_fn(g, mA, mB)
} by {
    if is_measurable_fn(f, mA, mB) and forall(x: A) { f(x) = g(x) } {
        is_measurable_fn(f, mA, mB) = forall(t: Set[B]) {
            mB.is_measurable(t) implies mA.is_measurable(set_preimage(f, t))
        }
        forall(t: Set[B]) {
            if mB.is_measurable(t) {
                forall(x: A) {
                    set_preimage_contains_eq(f, t, x)
                    set_preimage_contains_eq(g, t, x)
                    set_preimage(f, t).contains(x) = set_preimage(g, t).contains(x)
                }
                set_ext(set_preimage(f, t), set_preimage(g, t))
                mA.is_measurable(set_preimage(g, t))
            }
        }
    }
}

/// The constant function with value `c` is measurable for any sigma-algebras.
theorem is_measurable_fn_const[A, B](c: B,
        mA: SigmaAlgebra[A], mB: SigmaAlgebra[B]) {
    is_measurable_fn(function(x: A) { c }, mA, mB)
} by {
    forall(t: Set[B]) {
        if mB.is_measurable(t) {
            if t.contains(c) {
                forall(x: A) {
                    set_preimage_contains_eq(function(y: A) { c }, t, x)
                    function(y: A) { c }(x) = c
                    t.contains(c)
                    universal_set_contains_eq[A](x)
                    set_preimage(function(y: A) { c }, t).contains(x) =
                        Set[A].universal_set.contains(x)
                }
                set_ext(set_preimage(function(y: A) { c }, t),
                    Set[A].universal_set)
                sigma_algebra_measurable_universal(mA)
                mA.is_measurable(set_preimage(function(y: A) { c }, t))
            }
            if not t.contains(c) {
                forall(x: A) {
                    set_preimage_contains_eq(function(y: A) { c }, t, x)
                    function(y: A) { c }(x) = c
                    set_preimage(function(y: A) { c }, t).contains(x) =
                        t.contains(c)
                    empty_set_contains_eq[A](x)
                    set_preimage(function(y: A) { c }, t).contains(x) =
                        Set[A].empty_set.contains(x)
                }
                set_ext(set_preimage(function(y: A) { c }, t),
                    Set[A].empty_set)
                sigma_algebra_measurable_empty(mA)
                mA.is_measurable(set_preimage(function(y: A) { c }, t))
            }
            mA.is_measurable(set_preimage(function(y: A) { c }, t))
        }
    }
}

/// True if `s` is the preimage under `f` of some `mB`-measurable subset.
define is_preimage_measurable[A, B](f: A -> B, mB: SigmaAlgebra[B],
        s: Set[A]) -> Bool {
    exists(t: Set[B]) {
        mB.is_measurable(t) and s = set_preimage(f, t)
    }
}

/// The preimage measurable predicate contains the empty set.
theorem preimage_measurable_empty[A, B](f: A -> B, mB: SigmaAlgebra[B]) {
    has_measurable_empty_constraint(
        function(s: Set[A]) { is_preimage_measurable(f, mB, s) })
} by {
    sigma_algebra_measurable_empty(mB)
    set_preimage_empty(f)
    is_preimage_measurable(f, mB, Set[A].empty_set)
    function(s: Set[A]) { is_preimage_measurable(f, mB, s) }(Set[A].empty_set) =
        is_preimage_measurable(f, mB, Set[A].empty_set)
}

/// The preimage measurable predicate is closed under complement.
theorem preimage_measurable_compl[A, B](f: A -> B, mB: SigmaAlgebra[B]) {
    closed_under_compl_constraint(
        function(s: Set[A]) { is_preimage_measurable(f, mB, s) })
} by {
    forall(s: Set[A]) {
        if function(u: Set[A]) { is_preimage_measurable(f, mB, u) }(s) {
            let t: Set[B] satisfy {
                mB.is_measurable(t) and s = set_preimage(f, t)
            }
            sigma_algebra_measurable_compl(mB, t)
            set_preimage_compl(f, t)
            s.c = set_preimage(f, t.c)
            function(u: Set[A]) { is_preimage_measurable(f, mB, u) }(s.c)
        }
    }
}

/// The preimage measurable predicate is closed under countable union.
theorem preimage_measurable_indexed_union[A, B](f: A -> B,
        mB: SigmaAlgebra[B]) {
    closed_under_countable_union_constraint(
        function(s: Set[A]) { is_preimage_measurable(f, mB, s) })
} by {
    forall(family: Nat -> Set[A]) {
        if forall(n: Nat) {
            function(u: Set[A]) { is_preimage_measurable(f, mB, u) }(family(n))
        } {
            forall(n: Nat) {
                is_preimage_measurable(f, mB, family(n))
            }
            let pick: Nat -> Set[B] = function(n: Nat) {
                choose_or_default(
                    function(u: Set[B]) {
                        mB.is_measurable(u) and family(n) = set_preimage(f, u)
                    },
                    Set[B].empty_set)
            }
            forall(n: Nat) {
                is_preimage_measurable(f, mB, family(n))
                exists(u: Set[B]) {
                    mB.is_measurable(u) and family(n) = set_preimage(f, u)
                }
                choose_or_default_spec(
                    function(u: Set[B]) {
                        mB.is_measurable(u) and family(n) = set_preimage(f, u)
                    },
                    Set[B].empty_set)
                function(u: Set[B]) {
                    mB.is_measurable(u) and family(n) = set_preimage(f, u)
                }(pick(n))
                mB.is_measurable(pick(n)) and family(n) = set_preimage(f, pick(n))
            }
            forall(n: Nat) {
                mB.is_measurable(pick(n))
            }
            forall(n: Nat) {
                family(n) = set_preimage(f, pick(n))
            }
            sigma_algebra_measurable_indexed_union(mB, pick)
            set_preimage_indexed_union(f, pick)
            indexed_union(function(i: Nat) { family(i) }) =
                indexed_union(function(i: Nat) { set_preimage(f, pick(i) )})
            family = function(i: Nat) { family(i) }
            indexed_union(family) =
                indexed_union(function(i: Nat) { family(i) })
            set_preimage(f, indexed_union(pick)) = indexed_union(family)
            function(u: Set[A]) {
                is_preimage_measurable(f, mB, u)
            }(indexed_union(family))
        }
    }
}

/// The preimage measurable predicate satisfies the sigma-algebra axioms.
theorem preimage_measurable_is_sigma_algebra[A, B](f: A -> B,
        mB: SigmaAlgebra[B]) {
    has_measurable_empty_constraint(
        function(s: Set[A]) { is_preimage_measurable(f, mB, s) })
        and closed_under_compl_constraint(
            function(s: Set[A]) { is_preimage_measurable(f, mB, s) })
        and closed_under_countable_union_constraint(
            function(s: Set[A]) { is_preimage_measurable(f, mB, s) })
} by {
    preimage_measurable_empty(f, mB)
    preimage_measurable_compl(f, mB)
    preimage_measurable_indexed_union(f, mB)
}

/// The preimage sigma-algebra on `A` induced by `f: A -> B` and the
/// sigma-algebra `mB` on `B`: a set is measurable when it is the preimage
/// of some `mB`-measurable subset of `B`.
let preimage_sigma_algebra[A, B](f: A -> B, mB: SigmaAlgebra[B]) -> result: SigmaAlgebra[A] satisfy {
    SigmaAlgebra.new(function(s: Set[A]) { is_preimage_measurable(f, mB, s) }) = Option.some(result)
} by {
    preimage_measurable_is_sigma_algebra(f, mB)
}

/// The preimage sigma-algebra measures exactly the preimages of measurable sets.
theorem preimage_sigma_algebra_is_measurable_eq[A, B](f: A -> B,
        mB: SigmaAlgebra[B], s: Set[A]) {
    preimage_sigma_algebra(f, mB).is_measurable(s) =
        is_preimage_measurable(f, mB, s)
} by {
    SigmaAlgebra.new(function(u: Set[A]) { is_preimage_measurable(f, mB, u) }) =
        Option.some(preimage_sigma_algebra(f, mB))
}

/// Every preimage of an `mB`-measurable set lies in the preimage sigma-algebra.
theorem preimage_sigma_algebra_is_measurable_preimage[A, B](f: A -> B,
        mB: SigmaAlgebra[B], t: Set[B]) {
    mB.is_measurable(t)
        implies preimage_sigma_algebra(f, mB).is_measurable(set_preimage(f, t))
} by {
    if mB.is_measurable(t) {
        is_preimage_measurable(f, mB, set_preimage(f, t))
        preimage_sigma_algebra_is_measurable_eq(f, mB, set_preimage(f, t))
        preimage_sigma_algebra(f, mB).is_measurable(set_preimage(f, t))
    }
}

/// `f` is measurable from its preimage sigma-algebra to `mB`.
theorem preimage_sigma_algebra_makes_fn_measurable[A, B](f: A -> B,
        mB: SigmaAlgebra[B]) {
    is_measurable_fn(f, preimage_sigma_algebra(f, mB), mB)
} by {
    forall(t: Set[B]) {
        if mB.is_measurable(t) {
            preimage_sigma_algebra_is_measurable_preimage(f, mB, t)
        }
    }
}

/// The preimage sigma-algebra is the coarsest sigma-algebra on `A` making
/// `f` measurable: any sigma-algebra `mA` making `f` measurable refines it.
theorem preimage_sigma_algebra_is_coarsest[A, B](f: A -> B,
        mA: SigmaAlgebra[A], mB: SigmaAlgebra[B]) {
    is_measurable_fn(f, mA, mB)
        implies sigma_algebra_le(preimage_sigma_algebra(f, mB), mA)
} by {
    if is_measurable_fn(f, mA, mB) {
        forall(s: Set[A]) {
            if preimage_sigma_algebra(f, mB).is_measurable(s) {
                preimage_sigma_algebra_is_measurable_eq(f, mB, s)
                let t: Set[B] satisfy {
                    mB.is_measurable(t) and s = set_preimage(f, t)
                }
                is_measurable_fn(f, mA, mB) = forall(u: Set[B]) {
                    mB.is_measurable(u) implies mA.is_measurable(set_preimage(f, u))
                }
                mA.is_measurable(s)
            }
        }
    }
}

/// If `f` is measurable from `mA` and `mA` refines into `mD` on the same
/// domain, then `f` is still measurable when the domain sigma-algebra is
/// replaced by the finer one.
theorem is_measurable_fn_mono_domain[A, B](f: A -> B,
        mA: SigmaAlgebra[A], mD: SigmaAlgebra[A], mB: SigmaAlgebra[B]) {
    is_measurable_fn(f, mA, mB) and sigma_algebra_le(mA, mD)
        implies is_measurable_fn(f, mD, mB)
} by {
    if is_measurable_fn(f, mA, mB) and sigma_algebra_le(mA, mD) {
        is_measurable_fn(f, mA, mB) = forall(u: Set[B]) {
            mB.is_measurable(u) implies mA.is_measurable(set_preimage(f, u))
        }
        sigma_algebra_le(mA, mD) = forall(s: Set[A]) {
            mA.is_measurable(s) implies mD.is_measurable(s)
        }
        forall(t: Set[B]) {
            if mB.is_measurable(t) {
                mD.is_measurable(set_preimage(f, t))
            }
        }
    }
}

/// True if `s` is measurable in every sigma-algebra containing the generator `g`.
define generated_measurable[K](g: Set[K] -> Bool, s: Set[K]) -> Bool {
    forall(m: SigmaAlgebra[K]) {
        (forall(t: Set[K]) { g(t) implies m.is_measurable(t) })
            implies m.is_measurable(s)
    }
}

/// Instantiating the generated measurable predicate at a covering sigma-algebra.
theorem generated_measurable_apply[K](g: Set[K] -> Bool, s: Set[K],
        m: SigmaAlgebra[K]) {
    generated_measurable(g, s)
        and (forall(t: Set[K]) { g(t) implies m.is_measurable(t) })
        implies m.is_measurable(s)
} by {
    if generated_measurable(g, s)
            and forall(t: Set[K]) { g(t) implies m.is_measurable(t) } {
        generated_measurable(g, s) = forall(p: SigmaAlgebra[K]) {
            (forall(t: Set[K]) { g(t) implies p.is_measurable(t) })
                implies p.is_measurable(s)
        }
    }
}

/// The generated measurable predicate contains the empty set.
theorem generated_measurable_empty[K](g: Set[K] -> Bool) {
    has_measurable_empty_constraint(
        function(s: Set[K]) { generated_measurable(g, s) })
} by {
    forall(m: SigmaAlgebra[K]) {
        if forall(t: Set[K]) { g(t) implies m.is_measurable(t) } {
            sigma_algebra_measurable_empty(m)
            m.is_measurable(Set[K].empty_set)
        }
    }
    generated_measurable(g, Set[K].empty_set)
    function(s: Set[K]) { generated_measurable(g, s) }(Set[K].empty_set) =
        generated_measurable(g, Set[K].empty_set)
}

/// The generated measurable predicate is closed under complement.
theorem generated_measurable_compl[K](g: Set[K] -> Bool) {
    closed_under_compl_constraint(
        function(s: Set[K]) { generated_measurable(g, s) })
} by {
    forall(s: Set[K]) {
        if function(u: Set[K]) { generated_measurable(g, u) }(s) {
            forall(m: SigmaAlgebra[K]) {
                if forall(t: Set[K]) { g(t) implies m.is_measurable(t) } {
                    generated_measurable_apply(g, s, m)
                    sigma_algebra_measurable_compl(m, s)
                    m.is_measurable(s.c)
                }
            }
            function(u: Set[K]) { generated_measurable(g, u) }(s.c)
        }
    }
}

/// The generated measurable predicate is closed under countable union.
theorem generated_measurable_indexed_union[K](g: Set[K] -> Bool) {
    closed_under_countable_union_constraint(
        function(s: Set[K]) { generated_measurable(g, s) })
} by {
    forall(family: Nat -> Set[K]) {
        if forall(n: Nat) {
            function(u: Set[K]) { generated_measurable(g, u) }(family(n))
        } {
            forall(n: Nat) {
                generated_measurable(g, family(n))
            }
            forall(m: SigmaAlgebra[K]) {
                if forall(t: Set[K]) { g(t) implies m.is_measurable(t) } {
                    forall(n: Nat) {
                        generated_measurable_apply(g, family(n), m)
                        m.is_measurable(family(n))
                    }
                    sigma_algebra_measurable_indexed_union(m, family)
                    m.is_measurable(indexed_union(family))
                }
            }
            function(u: Set[K]) {
                generated_measurable(g, u)
            }(indexed_union(family))
        }
    }
}

/// The generated measurable predicate satisfies the sigma-algebra axioms.
theorem generated_measurable_is_sigma_algebra[K](g: Set[K] -> Bool) {
    has_measurable_empty_constraint(
        function(s: Set[K]) { generated_measurable(g, s) })
        and closed_under_compl_constraint(
            function(s: Set[K]) { generated_measurable(g, s) })
        and closed_under_countable_union_constraint(
            function(s: Set[K]) { generated_measurable(g, s) })
} by {
    generated_measurable_empty(g)
    generated_measurable_compl(g)
    generated_measurable_indexed_union(g)
}

/// True if `t` has `mA`-measurable preimage under `f`.
define pullback_measurable[A, B](f: A -> B, mA: SigmaAlgebra[A], t: Set[B]) -> Bool {
    mA.is_measurable(set_preimage(f, t))
}

/// The pullback measurable predicate contains the empty set.
theorem pullback_measurable_empty[A, B](f: A -> B, mA: SigmaAlgebra[A]) {
    has_measurable_empty_constraint(
        function(t: Set[B]) { pullback_measurable(f, mA, t) })
} by {
    set_preimage_empty(f)
    sigma_algebra_measurable_empty(mA)
    function(t: Set[B]) { pullback_measurable(f, mA, t) }(Set[B].empty_set) =
        pullback_measurable(f, mA, Set[B].empty_set)
}

/// The pullback measurable predicate is closed under complement.
theorem pullback_measurable_compl[A, B](f: A -> B, mA: SigmaAlgebra[A]) {
    closed_under_compl_constraint(
        function(t: Set[B]) { pullback_measurable(f, mA, t) })
} by {
    forall(t: Set[B]) {
        if function(u: Set[B]) { pullback_measurable(f, mA, u) }(t) {
            sigma_algebra_measurable_compl(mA, set_preimage(f, t))
            mA.is_measurable(set_preimage(f, t).c)
            set_preimage_compl(f, t)
            function(u: Set[B]) { pullback_measurable(f, mA, u) }(t.c)
        }
    }
}

/// The pullback measurable predicate is closed under countable union.
theorem pullback_measurable_indexed_union[A, B](f: A -> B, mA: SigmaAlgebra[A]) {
    closed_under_countable_union_constraint(
        function(t: Set[B]) { pullback_measurable(f, mA, t) })
} by {
    forall(family: Nat -> Set[B]) {
        if forall(n: Nat) {
            function(u: Set[B]) { pullback_measurable(f, mA, u) }(family(n))
        } {
            forall(n: Nat) {
                mA.is_measurable(set_preimage(f, family(n)))
            }
            let preimg_family: Nat -> Set[A] =
                function(n: Nat) { set_preimage(f, family(n)) }
            forall(n: Nat) {
                mA.is_measurable(preimg_family(n))
            }
            sigma_algebra_measurable_indexed_union(mA, preimg_family)
            mA.is_measurable(indexed_union(preimg_family))
            set_preimage_indexed_union(f, family)
            indexed_union(preimg_family) =
                indexed_union(function(i: Nat) { set_preimage(f, family(i)) })
            function(u: Set[B]) { pullback_measurable(f, mA, u) }(
                indexed_union(family))
        }
    }
}

/// The pullback measurable predicate satisfies the sigma-algebra axioms.
theorem pullback_measurable_is_sigma_algebra[A, B](f: A -> B,
        mA: SigmaAlgebra[A]) {
    has_measurable_empty_constraint(
        function(t: Set[B]) { pullback_measurable(f, mA, t) })
        and closed_under_compl_constraint(
            function(t: Set[B]) { pullback_measurable(f, mA, t) })
        and closed_under_countable_union_constraint(
            function(t: Set[B]) { pullback_measurable(f, mA, t) })
} by {
    pullback_measurable_empty(f, mA)
    pullback_measurable_compl(f, mA)
    pullback_measurable_indexed_union(f, mA)
}

/// The pullback sigma-algebra on `B` induced by `f: A -> B` and a
/// sigma-algebra `mA` on `A`: a set is measurable when its preimage under
/// `f` is `mA`-measurable. This is the finest sigma-algebra on `B` making
/// `f` measurable.
let pullback_sigma_algebra[A, B](f: A -> B, mA: SigmaAlgebra[A]) -> result: SigmaAlgebra[B] satisfy {
    SigmaAlgebra.new(function(t: Set[B]) { pullback_measurable(f, mA, t) }) = Option.some(result)
} by {
    pullback_measurable_is_sigma_algebra(f, mA)
}

/// The pullback sigma-algebra measures exactly the sets whose preimage is
/// `mA`-measurable.
theorem pullback_sigma_algebra_is_measurable_eq[A, B](f: A -> B,
        mA: SigmaAlgebra[A], t: Set[B]) {
    pullback_sigma_algebra(f, mA).is_measurable(t) = pullback_measurable(f, mA, t)
} by {
    SigmaAlgebra.new(function(u: Set[B]) { pullback_measurable(f, mA, u) }) =
        Option.some(pullback_sigma_algebra(f, mA))
}

/// `f` is measurable from `mA` to the pullback sigma-algebra.
theorem pullback_sigma_algebra_makes_fn_measurable[A, B](f: A -> B,
        mA: SigmaAlgebra[A]) {
    is_measurable_fn(f, mA, pullback_sigma_algebra(f, mA))
} by {
    forall(t: Set[B]) {
        if pullback_sigma_algebra(f, mA).is_measurable(t) {
            pullback_sigma_algebra_is_measurable_eq(f, mA, t)
            mA.is_measurable(set_preimage(f, t))
        }
    }
}

/// The sigma-algebra generated by a collection of sets `g`: the smallest
/// sigma-algebra in which every set satisfying `g` is measurable.
let generated_sigma_algebra[K](g: Set[K] -> Bool) -> result: SigmaAlgebra[K] satisfy {
    SigmaAlgebra.new(function(s: Set[K]) { generated_measurable(g, s) }) = Option.some(result)
} by {
    generated_measurable_is_sigma_algebra(g)
}

/// A set is measurable in the generated sigma-algebra exactly when it
/// satisfies the generated measurable predicate.
theorem generated_sigma_algebra_is_measurable_eq[K](g: Set[K] -> Bool, s: Set[K]) {
    generated_sigma_algebra(g).is_measurable(s) = generated_measurable(g, s)
} by {
}

/// Every generator set lies in the generated sigma-algebra.
theorem generated_sigma_algebra_contains_generator[K](g: Set[K] -> Bool, s: Set[K]) {
    g(s) implies generated_sigma_algebra(g).is_measurable(s)
} by {
    if g(s) {
        forall(m: SigmaAlgebra[K]) {
            if forall(t: Set[K]) { g(t) implies m.is_measurable(t) } {
                m.is_measurable(s)
            }
        }
        generated_measurable(g, s)
        generated_sigma_algebra_is_measurable_eq(g, s)
        generated_sigma_algebra(g).is_measurable(s)
    }
}

/// The generated sigma-algebra is the smallest: any sigma-algebra containing
/// the generator refines it.
theorem generated_sigma_algebra_is_smallest[K](g: Set[K] -> Bool, m: SigmaAlgebra[K]) {
    (forall(t: Set[K]) { g(t) implies m.is_measurable(t) })
        implies sigma_algebra_le(generated_sigma_algebra(g), m)
} by {
    if forall(t: Set[K]) { g(t) implies m.is_measurable(t) } {
        forall(s: Set[K]) {
            if generated_sigma_algebra(g).is_measurable(s) {
                generated_sigma_algebra_is_measurable_eq(g, s)
                generated_measurable_apply(g, s, m)
                m.is_measurable(s)
            }
        }
    }
}

/// A function is measurable into a generated sigma-algebra when the
/// preimage of every generator is measurable.
theorem is_measurable_fn_into_generated[A, B](f: A -> B, mA: SigmaAlgebra[A],
        g: Set[B] -> Bool) {
    (forall(t: Set[B]) { g(t) implies mA.is_measurable(set_preimage(f, t)) })
        implies is_measurable_fn(f, mA, generated_sigma_algebra(g))
} by {
    if forall(t: Set[B]) { g(t) implies mA.is_measurable(set_preimage(f, t)) } {
        forall(t: Set[B]) {
            if g(t) {
                pullback_sigma_algebra_is_measurable_eq(f, mA, t)
                pullback_sigma_algebra(f, mA).is_measurable(t)
            }
        }
        generated_sigma_algebra_is_smallest(g, pullback_sigma_algebra[A, B](f, mA))
        sigma_algebra_le(generated_sigma_algebra(g), pullback_sigma_algebra(f, mA))
        sigma_algebra_le(generated_sigma_algebra(g), pullback_sigma_algebra(f, mA)) =
            forall(s: Set[B]) {
                generated_sigma_algebra(g).is_measurable(s)
                    implies pullback_sigma_algebra(f, mA).is_measurable(s)
            }
        forall(t: Set[B]) {
            if generated_sigma_algebra(g).is_measurable(t) {
                pullback_sigma_algebra_is_measurable_eq(f, mA, t)
                pullback_measurable(f, mA, t)
                mA.is_measurable(set_preimage(f, t))
            }
        }
        is_measurable_fn(f, mA, generated_sigma_algebra(g))
    }
}

/// The generated sigma-algebra is monotone in its generator.
theorem generated_sigma_algebra_mono[K](g1: Set[K] -> Bool, g2: Set[K] -> Bool) {
    (forall(s: Set[K]) { g1(s) implies g2(s) })
        implies sigma_algebra_le(generated_sigma_algebra(g1),
            generated_sigma_algebra(g2))
} by {
    if forall(s: Set[K]) { g1(s) implies g2(s) } {
        forall(t: Set[K]) {
            if g1(t) {
                generated_sigma_algebra_contains_generator(g2, t)
                generated_sigma_algebra(g2).is_measurable(t)
            }
        }
        generated_sigma_algebra_is_smallest(g1, generated_sigma_algebra(g2))
    }
}

/// Refinement into the generated sigma-algebra: it suffices to check the
/// generator.
theorem generated_sigma_algebra_le_iff[K](g: Set[K] -> Bool, m: SigmaAlgebra[K]) {
    sigma_algebra_le(generated_sigma_algebra(g), m) iff
        forall(t: Set[K]) { g(t) implies m.is_measurable(t) }
} by {
    if sigma_algebra_le(generated_sigma_algebra(g), m) {
        sigma_algebra_le(generated_sigma_algebra(g), m) =
            forall(s: Set[K]) {
                generated_sigma_algebra(g).is_measurable(s) implies m.is_measurable(s)
            }
        forall(t: Set[K]) {
            if g(t) {
                generated_sigma_algebra_contains_generator(g, t)
                m.is_measurable(t)
            }
        }
    }
    if forall(t: Set[K]) { g(t) implies m.is_measurable(t) } {
        generated_sigma_algebra_is_smallest(g, m)
    }
}

/// A function is measurable into the generated sigma-algebra exactly when
/// the preimage of every generator is measurable.
theorem is_measurable_fn_into_generated_iff[A, B](f: A -> B,
        mA: SigmaAlgebra[A], g: Set[B] -> Bool) {
    is_measurable_fn(f, mA, generated_sigma_algebra(g)) iff
        forall(t: Set[B]) { g(t) implies mA.is_measurable(set_preimage(f, t)) }
} by {
    if is_measurable_fn(f, mA, generated_sigma_algebra(g)) {
        is_measurable_fn(f, mA, generated_sigma_algebra(g)) = forall(u: Set[B]) {
            generated_sigma_algebra(g).is_measurable(u)
                implies mA.is_measurable(set_preimage(f, u))
        }
        forall(t: Set[B]) {
            if g(t) {
                generated_sigma_algebra_contains_generator(g, t)
                mA.is_measurable(set_preimage(f, t))
            }
        }
    }
    if forall(t: Set[B]) { g(t) implies mA.is_measurable(set_preimage(f, t)) } {
        is_measurable_fn_into_generated(f, mA, g)
    }
}

/// The generator on the domain associated to a generator on the codomain:
/// a set `s` of `A` is included exactly when it is the preimage under `f`
/// of some `g`-set of `B`.
define preimage_generator[A, B](f: A -> B, g: Set[B] -> Bool) -> (Set[A] -> Bool) {
    function(s: Set[A]) {
        exists(t: Set[B]) { g(t) and s = set_preimage(f, t) }
    }
}

/// Preimage commutes with generation: the preimage sigma-algebra of a
/// generated sigma-algebra is generated by the preimages of the generators.
theorem preimage_sigma_algebra_generated_eq[A, B](f: A -> B, g: Set[B] -> Bool) {
    preimage_sigma_algebra(f, generated_sigma_algebra(g))
        = generated_sigma_algebra(preimage_generator(f, g))
} by {
    let m1: SigmaAlgebra[A] = preimage_sigma_algebra(f, generated_sigma_algebra(g))
    let h: Set[A] -> Bool = preimage_generator(f, g)
    let m2: SigmaAlgebra[A] = generated_sigma_algebra(h)
    forall(t: Set[B]) {
        if g(t) {
            h(set_preimage(f, t)) =
                exists(u: Set[B]) { g(u) and set_preimage(f, t) = set_preimage(f, u) }
            generated_sigma_algebra_contains_generator(h, set_preimage(f, t))
            m2.is_measurable(set_preimage(f, t))
        }
    }
    is_measurable_fn_into_generated(f, m2, g)
    is_measurable_fn(f, m2, generated_sigma_algebra(g))
    preimage_sigma_algebra_is_coarsest(f, m2, generated_sigma_algebra(g))
    sigma_algebra_le(m1, m2)
    forall(s: Set[A]) {
        if h(s) {
            let t: Set[B] satisfy {
                g(t) and s = set_preimage(f, t)
            }
            generated_sigma_algebra_contains_generator(g, t)
            preimage_sigma_algebra_is_measurable_preimage(f, generated_sigma_algebra(g), t)
            m1.is_measurable(set_preimage(f, t))
            m1.is_measurable(s)
        }
    }
    generated_sigma_algebra_is_smallest(h, m1)
    sigma_algebra_le(m2, m1)
    sigma_algebra_le_antisymm(m1, m2)
}

/// If every measurable set of `m` is in the generator, then the generated
/// sigma-algebra refines into `m`.
theorem generated_sigma_algebra_eq_of_generated_by_self[K](m: SigmaAlgebra[K]) {
    sigma_algebra_le(m,
        generated_sigma_algebra(function(s: Set[K]) { m.is_measurable(s) }))
} by {
    let g: Set[K] -> Bool = function(s: Set[K]) { m.is_measurable(s) }
    forall(s: Set[K]) {
        if m.is_measurable(s) {
            generated_sigma_algebra_contains_generator(g, s)
            generated_sigma_algebra(g).is_measurable(s)
        }
    }
}

/// The indicator function of a set: `x` maps to `true` when `x` is in `s`,
/// and to `false` otherwise.
define indicator_fn[T](s: Set[T]) -> (T -> Bool) {
    function(x: T) { s.contains(x) }
}

/// The "positive part" of the preimage of `t` under the indicator function
/// of `s`: equals `s` when `true` is in `t`, and the empty set otherwise.
define indicator_preimage_pos[T](s: Set[T], t: Set[Bool]) -> Set[T] {
    if t.contains(true) { s } else { Set[T].empty_set }
}

/// The "negative part" of the preimage of `t` under the indicator function
/// of `s`: equals the complement of `s` when `false` is in `t`, and the empty
/// set otherwise.
define indicator_preimage_neg[T](s: Set[T], t: Set[Bool]) -> Set[T] {
    if t.contains(false) { s.c } else { Set[T].empty_set }
}

/// The preimage of `t` under the indicator function of `s` decomposes as the
/// union of the positive and negative parts.
theorem indicator_fn_preimage_eq[T](s: Set[T], t: Set[Bool]) {
    set_preimage(indicator_fn(s), t) =
        indicator_preimage_pos(s, t).union(indicator_preimage_neg(s, t))
} by {
    let ind: T -> Bool = indicator_fn(s)
    let a: Set[T] = indicator_preimage_pos(s, t)
    let b: Set[T] = indicator_preimage_neg(s, t)
    forall(x: T) {
        set_preimage_contains_eq(ind, t, x)
        ind(x) = s.contains(x)
        set_preimage(ind, t).contains(x) = t.contains(s.contains(x))
        union_contains_eq(a, b, x)
        if s.contains(x) {
            t.contains(s.contains(x)) = t.contains(true)
            if t.contains(true) {
                a = s
                set_preimage(ind, t).contains(x) = a.union(b).contains(x)
            }
            if not t.contains(true) {
                a = Set[T].empty_set
                empty_set_contains_eq[T](x)
                not a.contains(x)
                not t.contains(s.contains(x))
                not set_preimage(ind, t).contains(x)
                if b.contains(x) {
                    if t.contains(false) {
                        b = s.c
                        compl_contains_eq(s, x)
                        false
                    }
                    if not t.contains(false) {
                        b = Set[T].empty_set
                        false
                    }
                }
                not b.contains(x)
                set_preimage(ind, t).contains(x) = a.union(b).contains(x)
            }
            set_preimage(ind, t).contains(x) = a.union(b).contains(x)
        }
        if not s.contains(x) {
            t.contains(s.contains(x)) = t.contains(false)
            compl_contains_eq(s, x)
            s.c.contains(x)
            if t.contains(false) {
                b = s.c
                set_preimage(ind, t).contains(x) = a.union(b).contains(x)
            }
            if not t.contains(false) {
                b = Set[T].empty_set
                empty_set_contains_eq[T](x)
                not b.contains(x)
                not t.contains(s.contains(x))
                not set_preimage(ind, t).contains(x)
                if a.contains(x) {
                    if t.contains(true) {
                        a = s
                        false
                    }
                    if not t.contains(true) {
                        a = Set[T].empty_set
                        false
                    }
                }
                not a.contains(x)
                set_preimage(ind, t).contains(x) = a.union(b).contains(x)
            }
            set_preimage(ind, t).contains(x) = a.union(b).contains(x)
        }
        set_preimage(ind, t).contains(x) = a.union(b).contains(x)
    }
    set_ext(set_preimage(ind, t), a.union(b))
}

/// The positive part of an indicator preimage is measurable when `s` is.
theorem indicator_preimage_pos_is_measurable[T](s: Set[T], t: Set[Bool],
        mT: SigmaAlgebra[T]) {
    mT.is_measurable(s) implies mT.is_measurable(indicator_preimage_pos(s, t))
} by {
    if mT.is_measurable(s) {
        if t.contains(true) {
            indicator_preimage_pos(s, t) = s
            mT.is_measurable(indicator_preimage_pos(s, t))
        }
        if not t.contains(true) {
            indicator_preimage_pos(s, t) = Set[T].empty_set
            sigma_algebra_measurable_empty(mT)
            mT.is_measurable(indicator_preimage_pos(s, t))
        }
    }
}

/// The negative part of an indicator preimage is measurable when `s` is.
theorem indicator_preimage_neg_is_measurable[T](s: Set[T], t: Set[Bool],
        mT: SigmaAlgebra[T]) {
    mT.is_measurable(s) implies mT.is_measurable(indicator_preimage_neg(s, t))
} by {
    if mT.is_measurable(s) {
        if t.contains(false) {
            indicator_preimage_neg(s, t) = s.c
            sigma_algebra_measurable_compl(mT, s)
            mT.is_measurable(indicator_preimage_neg(s, t))
        }
        if not t.contains(false) {
            indicator_preimage_neg(s, t) = Set[T].empty_set
            sigma_algebra_measurable_empty(mT)
            mT.is_measurable(indicator_preimage_neg(s, t))
        }
    }
}

/// The indicator function of a measurable set is measurable into the discrete
/// sigma-algebra on `Bool`.
theorem indicator_fn_is_measurable_fn[T](s: Set[T], mT: SigmaAlgebra[T]) {
    mT.is_measurable(s) implies
        is_measurable_fn(indicator_fn(s), mT, SigmaAlgebra[Bool].discrete)
} by {
    if mT.is_measurable(s) {
        forall(t: Set[Bool]) {
            if SigmaAlgebra[Bool].discrete.is_measurable(t) {
                indicator_fn_preimage_eq(s, t)
                indicator_preimage_pos_is_measurable(s, t, mT)
                indicator_preimage_neg_is_measurable(s, t, mT)
                sigma_algebra_measurable_union(mT,
                    indicator_preimage_pos(s, t),
                    indicator_preimage_neg(s, t))
                mT.is_measurable(
                    indicator_preimage_pos(s, t).union(
                        indicator_preimage_neg(s, t)))
                mT.is_measurable(set_preimage(indicator_fn(s), t))
            }
        }
    }
}

/// The preimage of the singleton `{true}` under the indicator function of `s`
/// is exactly `s`.
theorem indicator_fn_preimage_true_eq[T](s: Set[T]) {
    set_preimage(indicator_fn(s),
        Set[Bool].new(function(b: Bool) { b })) = s
} by {
    let t: Set[Bool] = Set[Bool].new(function(b: Bool) { b })
    let ind: T -> Bool = indicator_fn(s)
    forall(b: Bool) {
        t.contains(b) = b
    }
    forall(x: T) {
        set_preimage_contains_eq(ind, t, x)
        set_preimage(ind, t).contains(x) = s.contains(x)
    }
    set_ext(set_preimage(ind, t), s)
}

/// If the indicator function of `s` is measurable into the discrete
/// sigma-algebra on `Bool`, then `s` is measurable.
theorem set_measurable_of_indicator_fn_measurable[T](s: Set[T],
        mT: SigmaAlgebra[T]) {
    is_measurable_fn(indicator_fn(s), mT, SigmaAlgebra[Bool].discrete)
        implies mT.is_measurable(s)
} by {
    if is_measurable_fn(indicator_fn(s), mT, SigmaAlgebra[Bool].discrete) {
        let t: Set[Bool] = Set[Bool].new(function(b: Bool) { b })
        discrete_sigma_algebra_is_measurable[Bool](t)
        is_measurable_fn(indicator_fn(s), mT, SigmaAlgebra[Bool].discrete) =
            forall(u: Set[Bool]) {
                SigmaAlgebra[Bool].discrete.is_measurable(u)
                    implies mT.is_measurable(set_preimage(indicator_fn(s), u))
            }
        indicator_fn_preimage_true_eq(s)
        set_preimage(indicator_fn(s), t) = s
        mT.is_measurable(s)
    }
}

/// The indicator function of `s` is measurable into the discrete sigma-algebra
/// on `Bool` if and only if `s` is measurable.
theorem indicator_fn_is_measurable_iff[T](s: Set[T], mT: SigmaAlgebra[T]) {
    is_measurable_fn(indicator_fn(s), mT, SigmaAlgebra[Bool].discrete) iff
        mT.is_measurable(s)
} by {
    if is_measurable_fn(indicator_fn(s), mT, SigmaAlgebra[Bool].discrete) {
        set_measurable_of_indicator_fn_measurable(s, mT)
    }
    if mT.is_measurable(s) {
        indicator_fn_is_measurable_fn(s, mT)
    }
}

/// True if `u` is measurable in the subspace `a` of the ambient sigma-algebra `m`:
/// `u` is the intersection of some ambient measurable set with `a`.
define is_measurable_in_subspace[T](m: SigmaAlgebra[T], a: Set[T], u: Set[T]) -> Bool {
    exists(v: Set[T]) {
        m.is_measurable(v) and u = v.intersection(a)
    }
}

/// The empty set is measurable in any subspace.
theorem subspace_measurable_empty[T](m: SigmaAlgebra[T], a: Set[T]) {
    is_measurable_in_subspace(m, a, Set[T].empty_set)
} by {
    let e = Set[T].empty_set
    sigma_algebra_measurable_empty(m)
    forall(x: T) {
        intersection_contains_eq(e, a, x)
        empty_set_contains_eq[T](x)
        e.intersection(a).contains(x) = e.contains(x)
    }
    set_ext(e.intersection(a), e)
    e = e.intersection(a)
}

/// The whole subspace `a` is measurable in itself.
theorem subspace_measurable_self[T](m: SigmaAlgebra[T], a: Set[T]) {
    is_measurable_in_subspace(m, a, a)
} by {
    sigma_algebra_measurable_universal(m)
    let u = Set[T].universal_set
    forall(x: T) {
        intersection_contains_eq(u, a, x)
        universal_set_contains_eq[T](x)
        u.intersection(a).contains(x) = a.contains(x)
    }
    set_ext(u.intersection(a), a)
    a = u.intersection(a)
}



/// The intersection of `a` with any ambient measurable set is measurable in subspace `a`.
theorem subspace_measurable_from_ambient[T](m: SigmaAlgebra[T], a: Set[T], v: Set[T]) {
    m.is_measurable(v) implies is_measurable_in_subspace(m, a, v.intersection(a))
} by {
    if m.is_measurable(v) {
        exists(w: Set[T]) {
            m.is_measurable(w) and v.intersection(a) = w.intersection(a)
        }
    }
}

/// The subspace complement of a subspace-measurable set is subspace-measurable.
theorem subspace_measurable_compl[T](m: SigmaAlgebra[T], a: Set[T], u: Set[T]) {
    is_measurable_in_subspace(m, a, u)
        implies is_measurable_in_subspace(m, a, a.difference(u))
} by {
    if is_measurable_in_subspace(m, a, u) {
        let v: Set[T] satisfy {
            m.is_measurable(v) and u = v.intersection(a)
        }
        sigma_algebra_measurable_compl(m, v)
        forall(x: T) {
            difference_contains_eq(a, u, x)
            intersection_contains_eq(v, a, x)
            compl_contains_eq(v, x)
            intersection_contains_eq(v.c, a, x)
            let p: Bool = a.contains(x)
            let nv: Bool = v.c.contains(x)
            let qv: Bool = v.contains(x)
            nv = not qv
            u.contains(x) = (qv and p)
            a.difference(u).contains(x) = (p and not (qv and p))
            (p and not (qv and p)) = (nv and p)
            a.difference(u).contains(x) = v.c.intersection(a).contains(x)
        }
        set_ext(a.difference(u), v.c.intersection(a))
        exists(w: Set[T]) {
            m.is_measurable(w) and a.difference(u) = w.intersection(a)
        }
    }
}

/// The subspace union of two subspace-measurable sets is subspace-measurable.
theorem subspace_measurable_union[T](m: SigmaAlgebra[T], a: Set[T],
        u: Set[T], v: Set[T]) {
    is_measurable_in_subspace(m, a, u) and is_measurable_in_subspace(m, a, v)
        implies is_measurable_in_subspace(m, a, u.union(v))
} by {
    if is_measurable_in_subspace(m, a, u) and is_measurable_in_subspace(m, a, v) {
        let p: Set[T] satisfy {
            m.is_measurable(p) and u = p.intersection(a)
        }
        let q: Set[T] satisfy {
            m.is_measurable(q) and v = q.intersection(a)
        }
        sigma_algebra_measurable_union(m, p, q)
        forall(x: T) {
            union_contains_eq(u, v, x)
            intersection_contains_eq(p, a, x)
            intersection_contains_eq(q, a, x)
            union_contains_eq(p, q, x)
            intersection_contains_eq(p.union(q), a, x)
            let px: Bool = p.contains(x)
            let qx: Bool = q.contains(x)
            let ax: Bool = a.contains(x)
            ((px and ax) or (qx and ax)) = ((px or qx) and ax)
            u.union(v).contains(x) = p.union(q).intersection(a).contains(x)
        }
        set_ext(u.union(v), p.union(q).intersection(a))
        exists(w: Set[T]) {
            m.is_measurable(w) and u.union(v) = w.intersection(a)
        }
    }
}

/// The subspace intersection of two subspace-measurable sets is subspace-measurable.
theorem subspace_measurable_intersection[T](m: SigmaAlgebra[T], a: Set[T],
        u: Set[T], v: Set[T]) {
    is_measurable_in_subspace(m, a, u) and is_measurable_in_subspace(m, a, v)
        implies is_measurable_in_subspace(m, a, u.intersection(v))
} by {
    if is_measurable_in_subspace(m, a, u) and is_measurable_in_subspace(m, a, v) {
        let p: Set[T] satisfy {
            m.is_measurable(p) and u = p.intersection(a)
        }
        let q: Set[T] satisfy {
            m.is_measurable(q) and v = q.intersection(a)
        }
        sigma_algebra_measurable_intersection(m, p, q)
        forall(x: T) {
            intersection_contains_eq(u, v, x)
            intersection_contains_eq(p, a, x)
            intersection_contains_eq(q, a, x)
            intersection_contains_eq(p, q, x)
            intersection_contains_eq(p.intersection(q), a, x)
            let px: Bool = p.contains(x)
            let qx: Bool = q.contains(x)
            let ax: Bool = a.contains(x)
            ((px and ax) and (qx and ax)) = ((px and qx) and ax)
            u.intersection(v).contains(x) =
                p.intersection(q).intersection(a).contains(x)
        }
        set_ext(u.intersection(v), p.intersection(q).intersection(a))
        exists(w: Set[T]) {
            m.is_measurable(w) and u.intersection(v) = w.intersection(a)
        }
    }
}

/// A subspace-measurable set is contained in its subspace.
theorem subspace_measurable_subset[T](m: SigmaAlgebra[T], a: Set[T], u: Set[T]) {
    is_measurable_in_subspace(m, a, u) implies u.subset(a)
} by {
    if is_measurable_in_subspace(m, a, u) {
        let v: Set[T] satisfy {
            m.is_measurable(v) and u = v.intersection(a)
        }
        forall(x: T) {
            if u.contains(x) {
                intersection_contains_eq(v, a, x)
                a.contains(x)
            }
        }
    }
}

/// The set difference of two subspace-measurable sets is subspace-measurable.
theorem subspace_measurable_difference[T](m: SigmaAlgebra[T], a: Set[T],
        u: Set[T], v: Set[T]) {
    is_measurable_in_subspace(m, a, u) and is_measurable_in_subspace(m, a, v)
        implies is_measurable_in_subspace(m, a, u.difference(v))
} by {
    if is_measurable_in_subspace(m, a, u) and is_measurable_in_subspace(m, a, v) {
        subspace_measurable_subset(m, a, u)
        u.subset(a) = forall(y: T) { u.contains(y) implies a.contains(y) }
        forall(y: T) { u.contains(y) implies a.contains(y) }
        subspace_measurable_compl(m, a, v)
        subspace_measurable_intersection(m, a, u, a.difference(v))
        is_measurable_in_subspace(m, a, u.intersection(a.difference(v)))
        forall(x: T) {
            difference_contains_eq(u, v, x)
            difference_contains_eq(a, v, x)
            intersection_contains_eq(u, a.difference(v), x)
            let ux: Bool = u.contains(x)
            let vx: Bool = v.contains(x)
            let ax: Bool = a.contains(x)
            (ux and not vx) = (ux and (ax and not vx))
            u.difference(v).contains(x) = u.intersection(a.difference(v)).contains(x)
        }
        set_ext(u.difference(v), u.intersection(a.difference(v)))
        let w: Set[T] satisfy {
            m.is_measurable(w)
                and u.intersection(a.difference(v)) = w.intersection(a)
        }
        exists(w2: Set[T]) {
            m.is_measurable(w2) and u.difference(v) = w2.intersection(a)
        }
    }
}

/// A countable union of subspace-measurable sets is subspace-measurable.
theorem subspace_measurable_indexed_union[T](m: SigmaAlgebra[T], a: Set[T],
        family: Nat -> Set[T]) {
    (forall(n: Nat) { is_measurable_in_subspace(m, a, family(n)) })
        implies is_measurable_in_subspace(m, a, indexed_union(family))
} by {
    if forall(n: Nat) { is_measurable_in_subspace(m, a, family(n)) } {
        let pick: Nat -> Set[T] = function(n: Nat) {
            choose_or_default(
                function(v: Set[T]) {
                    m.is_measurable(v) and family(n) = v.intersection(a)
                },
                Set[T].empty_set)
        }
        forall(n: Nat) {
            is_measurable_in_subspace(m, a, family(n))
            exists(v: Set[T]) {
                m.is_measurable(v) and family(n) = v.intersection(a)
            }
            choose_or_default_spec(
                function(v: Set[T]) {
                    m.is_measurable(v) and family(n) = v.intersection(a)
                },
                Set[T].empty_set)
            m.is_measurable(pick(n)) and family(n) = pick(n).intersection(a)
        }
        forall(n: Nat) {
            m.is_measurable(pick(n))
        }
        forall(n: Nat) {
            family(n) = pick(n).intersection(a)
        }
        sigma_algebra_measurable_indexed_union(m, pick)
        forall(x: T) {
            if indexed_union(pick).intersection(a).contains(x) {
                intersection_contains_eq(indexed_union(pick), a, x)
                indexed_union_contains_witness(pick, x)
                let n: Nat satisfy { pick(n).contains(x) }
                intersection_contains_eq(pick(n), a, x)
                indexed_union_contains_of_contains(family, n, x)
                indexed_union(family).contains(x)
            }
            if indexed_union(family).contains(x) {
                indexed_union_contains_witness(family, x)
                let n: Nat satisfy { family(n).contains(x) }
                intersection_contains_eq(pick(n), a, x)
                indexed_union_contains_of_contains(pick, n, x)
                indexed_union(pick).contains(x)
                intersection_contains_eq(indexed_union(pick), a, x)
                indexed_union(pick).intersection(a).contains(x)
            }
            indexed_union(pick).intersection(a).contains(x) =
                indexed_union(family).contains(x)
        }
        set_ext(indexed_union(pick).intersection(a), indexed_union(family))
        indexed_union(family) = indexed_union(pick).intersection(a)
        is_measurable_in_subspace(m, a, indexed_union(family))
    }
}

/// A countable intersection of subspace-measurable sets is subspace-measurable.
theorem subspace_measurable_indexed_intersection[T](m: SigmaAlgebra[T],
        a: Set[T], family: Nat -> Set[T]) {
    (forall(n: Nat) { is_measurable_in_subspace(m, a, family(n)) })
        implies is_measurable_in_subspace(m, a, indexed_intersection(family))
} by {
    if forall(n: Nat) { is_measurable_in_subspace(m, a, family(n)) } {
        let pick: Nat -> Set[T] = function(n: Nat) {
            choose_or_default(
                function(v: Set[T]) {
                    m.is_measurable(v) and family(n) = v.intersection(a)
                },
                Set[T].empty_set)
        }
        forall(n: Nat) {
            is_measurable_in_subspace(m, a, family(n))
            exists(v: Set[T]) {
                m.is_measurable(v) and family(n) = v.intersection(a)
            }
            choose_or_default_spec(
                function(v: Set[T]) {
                    m.is_measurable(v) and family(n) = v.intersection(a)
                },
                Set[T].empty_set)
            m.is_measurable(pick(n)) and family(n) = pick(n).intersection(a)
        }
        forall(n: Nat) {
            m.is_measurable(pick(n))
        }
        forall(n: Nat) {
            family(n) = pick(n).intersection(a)
        }
        sigma_algebra_measurable_indexed_intersection(m, pick)
        forall(x: T) {
            if indexed_intersection(pick).intersection(a).contains(x) {
                intersection_contains_eq(indexed_intersection(pick), a, x)
                forall(n: Nat) {
                    indexed_intersection_contains_at(pick, n, x)
                    intersection_contains_eq(pick(n), a, x)
                    pick(n).intersection(a).contains(x)
                    family(n).contains(x)
                }
                indexed_intersection_contains_of_forall(family, x)
                indexed_intersection(family).contains(x)
            }
            if indexed_intersection(family).contains(x) {
                indexed_intersection_contains_at(family, Nat.0, x)
                intersection_contains_eq(pick(Nat.0), a, x)
                forall(n: Nat) {
                    indexed_intersection_contains_at(family, n, x)
                    intersection_contains_eq(pick(n), a, x)
                    pick(n).contains(x)
                }
                indexed_intersection_contains_of_forall(pick, x)
                indexed_intersection(pick).contains(x)
                intersection_contains_eq(indexed_intersection(pick), a, x)
                indexed_intersection(pick).intersection(a).contains(x)
            }
            indexed_intersection(pick).intersection(a).contains(x) =
                indexed_intersection(family).contains(x)
        }
        set_ext(indexed_intersection(pick).intersection(a),
            indexed_intersection(family))
        indexed_intersection(family) = indexed_intersection(pick).intersection(a)
        is_measurable_in_subspace(m, a, indexed_intersection(family))
    }
}

/// The trace measurable predicate of `m` on subspace `a`: a set `u` is
/// trace-measurable when its intersection with `a` is measurable in the
/// subspace `a`.
define trace_measurable[T](m: SigmaAlgebra[T], a: Set[T], u: Set[T]) -> Bool {
    is_measurable_in_subspace(m, a, u.intersection(a))
}

/// The empty set is trace-measurable.
theorem trace_measurable_empty[T](m: SigmaAlgebra[T], a: Set[T]) {
    trace_measurable(m, a, Set[T].empty_set)
} by {
    let e = Set[T].empty_set
    forall(x: T) {
        intersection_contains_eq(e, a, x)
        empty_set_contains_eq[T](x)
        e.intersection(a).contains(x) = e.contains(x)
    }
    set_ext(e.intersection(a), e)
    subspace_measurable_empty(m, a)
}

/// The trace measurable predicate is closed under complement.
theorem trace_measurable_compl[T](m: SigmaAlgebra[T], a: Set[T], u: Set[T]) {
    trace_measurable(m, a, u) implies trace_measurable(m, a, u.c)
} by {
    if trace_measurable(m, a, u) {
        subspace_measurable_compl(m, a, u.intersection(a))
        is_measurable_in_subspace(m, a, a.difference(u.intersection(a)))
        forall(x: T) {
            difference_contains_eq(a, u.intersection(a), x)
            intersection_contains_eq(u, a, x)
            intersection_contains_eq(u.c, a, x)
            compl_contains_eq(u, x)
            if a.difference(u.intersection(a)).contains(x) {
                u.c.intersection(a).contains(x)
            }
            if u.c.intersection(a).contains(x) {
                a.difference(u.intersection(a)).contains(x)
            }
            a.difference(u.intersection(a)).contains(x) =
                u.c.intersection(a).contains(x)
        }
        set_ext(a.difference(u.intersection(a)), u.c.intersection(a))
        is_measurable_in_subspace(m, a, u.c.intersection(a))
    }
}

/// The trace measurable predicate is closed under countable union.
theorem trace_measurable_indexed_union[T](m: SigmaAlgebra[T], a: Set[T],
        family: Nat -> Set[T]) {
    (forall(n: Nat) { trace_measurable(m, a, family(n)) })
        implies trace_measurable(m, a, indexed_union(family))
} by {
    if forall(n: Nat) { trace_measurable(m, a, family(n)) } {
        let g: Nat -> Set[T] = function(n: Nat) { family(n).intersection(a) }
        forall(n: Nat) {
            is_measurable_in_subspace(m, a, g(n))
        }
        subspace_measurable_indexed_union(m, a, g)
        is_measurable_in_subspace(m, a, indexed_union(g))
        forall(x: T) {
            intersection_contains_eq(indexed_union(family), a, x)
            indexed_union_contains_eq(family, x)
            indexed_union_contains_eq(g, x)
            if indexed_union(g).contains(x) {
                indexed_union_contains_witness(g, x)
                let n: Nat satisfy { g(n).contains(x) }
                intersection_contains_eq(family(n), a, x)
                indexed_union_contains_of_contains(family, n, x)
                indexed_union(family).contains(x)
                indexed_union(family).intersection(a).contains(x)
            }
            if indexed_union(family).intersection(a).contains(x) {
                indexed_union_contains_witness(family, x)
                let n: Nat satisfy { family(n).contains(x) }
                intersection_contains_eq(family(n), a, x)
                indexed_union_contains_of_contains(g, n, x)
                indexed_union(g).contains(x)
            }
            indexed_union(g).contains(x) =
                indexed_union(family).intersection(a).contains(x)
        }
        set_ext(indexed_union(g), indexed_union(family).intersection(a))
        trace_measurable(m, a, indexed_union(family))
    }
}

/// The trace measurable predicate satisfies the sigma-algebra axioms.
theorem trace_measurable_is_sigma_algebra[T](m: SigmaAlgebra[T], a: Set[T]) {
    has_measurable_empty_constraint(
        function(u: Set[T]) { trace_measurable(m, a, u) })
        and closed_under_compl_constraint(
            function(u: Set[T]) { trace_measurable(m, a, u) })
        and closed_under_countable_union_constraint(
            function(u: Set[T]) { trace_measurable(m, a, u) })
} by {
    let p: Set[T] -> Bool = function(u: Set[T]) { trace_measurable(m, a, u) }
    forall(u: Set[T]) {
        p(u) = trace_measurable(m, a, u)
    }
    trace_measurable_empty(m, a)
    p(Set[T].empty_set) = trace_measurable(m, a, Set[T].empty_set)
    p(Set[T].empty_set)
    has_measurable_empty_constraint(p)
    forall(u: Set[T]) {
        if p(u) {
            trace_measurable_compl(m, a, u)
            p(u.c)
        }
    }
    closed_under_compl_constraint(p)
    forall(family: Nat -> Set[T]) {
        if forall(n: Nat) { p(family(n)) } {
            forall(n: Nat) {
                trace_measurable(m, a, family(n))
            }
            trace_measurable_indexed_union(m, a, family)
            p(indexed_union(family))
        }
    }
    closed_under_countable_union_constraint(p) = forall(family: Nat -> Set[T]) {
        (forall(n: Nat) { p(family(n)) }) implies p(indexed_union(family))
    }
    closed_under_countable_union_constraint(p)
}

/// The trace sigma-algebra of `m` on subspace `a`: a set is measurable when
/// its intersection with `a` is measurable in the subspace `a`.
let trace_sigma_algebra[T](m: SigmaAlgebra[T], a: Set[T]) -> result: SigmaAlgebra[T] satisfy {
    SigmaAlgebra.new(function(u: Set[T]) { trace_measurable(m, a, u) }) = Option.some(result)
} by {
    trace_measurable_is_sigma_algebra(m, a)
}

/// A set is measurable in the trace sigma-algebra exactly when its
/// intersection with `a` is measurable in the subspace `a`.
theorem trace_sigma_algebra_is_measurable_eq[T](m: SigmaAlgebra[T], a: Set[T],
        u: Set[T]) {
    trace_sigma_algebra(m, a).is_measurable(u) = trace_measurable(m, a, u)
} by {
}

/// For subsets of `a`, trace-measurability coincides with subspace-measurability.
theorem trace_sigma_algebra_measurable_iff_subspace[T](m: SigmaAlgebra[T],
        a: Set[T], u: Set[T]) {
    u.subset(a) implies
        (trace_sigma_algebra(m, a).is_measurable(u) iff
            is_measurable_in_subspace(m, a, u))
} by {
    if u.subset(a) {
        forall(x: T) {
            intersection_contains_eq(u, a, x)
            if u.intersection(a).contains(x) {
                u.contains(x)
            }
            if u.contains(x) {
                subset_contains(u, a, x)
                u.intersection(a).contains(x)
            }
            u.intersection(a).contains(x) = u.contains(x)
        }
        set_ext(u.intersection(a), u)
        trace_sigma_algebra_is_measurable_eq(m, a, u)
        is_measurable_in_subspace(m, a, u.intersection(a)) =
            is_measurable_in_subspace(m, a, u)
    }
}

/// Every subspace-measurable set is measurable in the trace sigma-algebra.
theorem trace_sigma_algebra_measurable_of_subspace[T](m: SigmaAlgebra[T], a: Set[T], u: Set[T]) {
    is_measurable_in_subspace(m, a, u) implies trace_sigma_algebra(m, a).is_measurable(u)
} by {
    if is_measurable_in_subspace(m, a, u) {
        subspace_measurable_subset(m, a, u)
        u.subset(a)
        trace_sigma_algebra_measurable_iff_subspace(m, a, u)
        trace_sigma_algebra(m, a).is_measurable(u)
    }
}

/// The indicator function of a subspace-measurable set is measurable from the
/// trace sigma-algebra into the discrete sigma-algebra on `Bool`.
theorem subspace_indicator_fn_is_measurable[T](m: SigmaAlgebra[T], a: Set[T],
        u: Set[T]) {
    is_measurable_in_subspace(m, a, u) implies
        is_measurable_fn(indicator_fn(u),
            trace_sigma_algebra(m, a), SigmaAlgebra[Bool].discrete)
} by {
    if is_measurable_in_subspace(m, a, u) {
        subspace_measurable_subset(m, a, u)
        trace_sigma_algebra_measurable_iff_subspace(m, a, u)
        trace_sigma_algebra(m, a).is_measurable(u)
        indicator_fn_is_measurable_fn(u, trace_sigma_algebra(m, a))
    }
}

/// If the indicator function of `u` is measurable from the trace sigma-algebra
/// into the discrete sigma-algebra on `Bool`, then `u` is trace-measurable.
theorem subspace_set_measurable_of_indicator_fn_measurable[T](m: SigmaAlgebra[T],
        a: Set[T], u: Set[T]) {
    is_measurable_fn(indicator_fn(u),
        trace_sigma_algebra(m, a), SigmaAlgebra[Bool].discrete)
        implies trace_sigma_algebra(m, a).is_measurable(u)
} by {
    if is_measurable_fn(indicator_fn(u),
        trace_sigma_algebra(m, a), SigmaAlgebra[Bool].discrete) {
        set_measurable_of_indicator_fn_measurable(u, trace_sigma_algebra(m, a))
    }
}

/// For subsets of `a`, the indicator function of `u` is measurable from the
/// trace sigma-algebra into the discrete sigma-algebra on `Bool` if and only if
/// `u` is measurable in the subspace `a`.
theorem subspace_indicator_fn_is_measurable_iff[T](m: SigmaAlgebra[T], a: Set[T],
        u: Set[T]) {
    u.subset(a) implies
        (is_measurable_fn(indicator_fn(u),
            trace_sigma_algebra(m, a), SigmaAlgebra[Bool].discrete) iff
            is_measurable_in_subspace(m, a, u))
} by {
    if u.subset(a) {
        trace_sigma_algebra_measurable_iff_subspace(m, a, u)
        indicator_fn_is_measurable_iff(u, trace_sigma_algebra(m, a))
        if is_measurable_fn(indicator_fn(u),
            trace_sigma_algebra(m, a), SigmaAlgebra[Bool].discrete) {
            is_measurable_in_subspace(m, a, u)
        }
        if is_measurable_in_subspace(m, a, u) {
            is_measurable_fn(indicator_fn(u),
                trace_sigma_algebra(m, a), SigmaAlgebra[Bool].discrete)
        }
    }
}
