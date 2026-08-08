from data.basic.set import Set, set_ext, subset_contains, subset_refl, subset_trans,
    subset_antisymm, empty_set_contains_eq, universal_set_contains_eq,
    union_contains_eq, intersection_contains_eq, difference_contains_eq,
    set_preimage, set_preimage_contains_eq, set_preimage_compl,
    preimage_contains, compl_contains_eq, compl_of_compl_is_self,
    empty_set_compl_is_universal, universal_set_compl_is_empty,
    is_saturated_under, empty_set_is_always_subset,
    sets_subset_intersection, sets_subset_union, sets_subset_contain_union,
    singleton_contains_eq, intersection_with_empty_is_empty,
    intersection_with_universal_is_self, intersection_comm, intersection_assoc,
    union_compl_is_universal, set_is_disjoint_compl
from data.basic.set_lattice import set_inf, set_subset_inf_of_subset_left_right
from data.basic.functions import identity_fn, compose
from pair import Pair, pair_new_first, pair_new_second, pair_ext
from sum_type import Sum
from list import List

/// Membership predicate for the union of a family of sets given by a predicate.
define big_union_contains[T](c: Set[T] -> Bool, x: T) -> Bool {
    exists(s: Set[T]) {
        c(s) and s.contains(x)
    }
}

/// The union of a family of sets, where the family is described by a predicate.
define big_union[T](c: Set[T] -> Bool) -> Set[T] {
    Set[T].new(big_union_contains(c))
}

/// Membership predicate for the intersection of a family of sets given by a predicate.
define big_intersection_contains[T](c: Set[T] -> Bool, x: T) -> Bool {
    forall(s: Set[T]) {
        c(s) implies s.contains(x)
    }
}

/// The intersection of a family of sets, where the family is described by a predicate.
define big_intersection[T](c: Set[T] -> Bool) -> Set[T] {
    Set[T].new(big_intersection_contains(c))
}

/// Membership in a big union is membership in some set of the family.
theorem big_union_contains_eq[T](c: Set[T] -> Bool, x: T) {
    big_union(c).contains(x) = exists(s: Set[T]) {
        c(s) and s.contains(x)
    }
} by {
    big_union(c).contains(x) = big_union_contains(c, x)
}

/// Membership of a witnessing set lifts to membership in the big union.
theorem big_union_contains_of_member[T](c: Set[T] -> Bool, s: Set[T], x: T) {
    c(s) and s.contains(x) implies big_union(c).contains(x)
} by {
    if c(s) and s.contains(x) {
        exists(t: Set[T]) {
            c(t) and t.contains(x)
        }
        big_union_contains_eq(c, x)
    }
}

/// Membership in a big intersection is membership in every set of the family.
theorem big_intersection_contains_eq[T](c: Set[T] -> Bool, x: T) {
    big_intersection(c).contains(x) = forall(s: Set[T]) {
        c(s) implies s.contains(x)
    }
} by {
    big_intersection(c).contains(x) = big_intersection_contains(c, x)
}

/// A topological space is a type equipped with a notion of open sets, where the empty
/// and whole space are open, finite intersections of open sets are open, and arbitrary
/// unions of open sets are open.
typeclass T: TopologicalSpace {
    /// True if the given subset is open in the topology.
    is_open: Set[T] -> Bool

    /// Rule: the empty set is open.
    open_empty {
        T.is_open(Set[T].empty_set)
    }

    /// Rule: the whole space is open.
    open_universal {
        T.is_open(Set[T].universal_set)
    }

    /// Rule: the intersection of two open sets is open.
    open_inter(s: Set[T], t: Set[T]) {
        T.is_open(s) and T.is_open(t) implies T.is_open(s.intersection(t))
    }

    /// Rule: the union of any family of open sets is open.
    open_big_union(c: Set[T] -> Bool) {
        (forall(s: Set[T]) { c(s) implies T.is_open(s) })
            implies T.is_open(big_union(c))
    }
}

/// The whole space is open.
theorem open_universal[T: TopologicalSpace] {
    T.is_open(Set[T].universal_set)
}

/// The intersection of two open sets is open.
theorem open_inter[T: TopologicalSpace](s: Set[T], t: Set[T]) {
    T.is_open(s) and T.is_open(t) implies T.is_open(s.intersection(t))
}

/// The union of any family of open sets is open.
theorem open_big_union[T: TopologicalSpace](c: Set[T] -> Bool) {
    (forall(s: Set[T]) { c(s) implies T.is_open(s) })
        implies T.is_open(big_union(c))
}

/// The empty set is open.
theorem open_empty[T: TopologicalSpace] {
    T.is_open(Set[T].empty_set)
}

/// True if a subset is closed: its complement is open.
define is_closed[T: TopologicalSpace](s: Set[T]) -> Bool {
    T.is_open(s.c)
}

/// The whole space is closed.
theorem closed_universal[T: TopologicalSpace] {
    is_closed(Set[T].universal_set)
} by {
    let u = Set[T].universal_set
    let e = Set[T].empty_set
    forall(x: T) {
        universal_set_contains_eq[T](x)
        empty_set_contains_eq[T](x)
        u.c.contains(x) = e.contains(x)
    }
    set_ext(u.c, e)
    u.c = e
    open_empty[T]
    T.is_open(u.c)
}

/// The empty set is closed.
theorem closed_empty[T: TopologicalSpace] {
    is_closed(Set[T].empty_set)
} by {
    let e = Set[T].empty_set
    let u = Set[T].universal_set
    forall(x: T) {
        empty_set_contains_eq[T](x)
        universal_set_contains_eq[T](x)
        e.c.contains(x) = u.contains(x)
    }
    set_ext(e.c, u)
    e.c = u
    open_universal[T]
    T.is_open(e.c)
}

/// Predicate for a pair family of two sets.
define pair_family[T](s: Set[T], t: Set[T], u: Set[T]) -> Bool {
    u = s or u = t
}

/// The big union of a pair family is the union of its two sets.
theorem big_union_pair[T](s: Set[T], t: Set[T]) {
    big_union(pair_family(s, t)) = s.union(t)
} by {
    let c = pair_family(s, t)
    let v = s.union(t)
    forall(x: T) {
        if big_union(c).contains(x) {
            big_union_contains_eq(c, x)
            let w: Set[T] satisfy {
                c(w) and w.contains(x)
            }
            pair_family(s, t, w) = (w = s or w = t)
            c(w) = (w = s or w = t)
            w = s or w = t
            if w = s {
                s.contains(x)
                union_contains_eq(s, t, x)
                v.contains(x)
            } else {
                w = t
                t.contains(x)
                union_contains_eq(s, t, x)
                v.contains(x)
            }
        }
        if v.contains(x) {
            union_contains_eq(s, t, x)
            if s.contains(x) {
                pair_family(s, t, s)
                c(s)
                big_union_contains_of_member(c, s, x)
                big_union(c).contains(x)
            } else {
                t.contains(x)
                pair_family(s, t, t)
                c(t)
                big_union_contains_of_member(c, t, x)
                big_union(c).contains(x)
            }
        }
        big_union(c).contains(x) = v.contains(x)
    }
    set_ext(big_union(c), v)
}

/// The big union of an everywhere-empty family is the empty set.
theorem big_union_of_empty_family[T](c: Set[T] -> Bool) {
    (forall(s: Set[T]) { c(s) implies s = Set[T].empty_set })
        implies big_union(c) = Set[T].empty_set
} by {
    if forall(s: Set[T]) { c(s) implies s = Set[T].empty_set } {
        forall(x: T) {
            empty_set_contains_eq[T](x)
            not Set[T].empty_set.contains(x)
            if big_union(c).contains(x) {
                big_union_contains_eq(c, x)
                let w: Set[T] satisfy {
                    c(w) and w.contains(x)
                }
                w = Set[T].empty_set
                Set[T].empty_set.contains(x)
                not Set[T].empty_set.contains(x)
                false
            }
            not big_union(c).contains(x)
            big_union(c).contains(x) = Set[T].empty_set.contains(x)
        }
        set_ext(big_union(c), Set[T].empty_set)
        big_union(c) = Set[T].empty_set
    }
}

/// If a family contains the universal set, its big union is the universal set.
theorem big_union_with_universal_member[T](c: Set[T] -> Bool) {
    c(Set[T].universal_set) implies big_union(c) = Set[T].universal_set
} by {
    if c(Set[T].universal_set) {
        forall(x: T) {
            universal_set_contains_eq[T](x)
            Set[T].universal_set.contains(x)
            big_union_contains_of_member(c, Set[T].universal_set, x)
            big_union(c).contains(x)
            big_union(c).contains(x) = Set[T].universal_set.contains(x)
        }
        set_ext(big_union(c), Set[T].universal_set)
        big_union(c) = Set[T].universal_set
    }
}

/// The union of two open sets is open.
theorem open_union_open[T: TopologicalSpace](s: Set[T], t: Set[T]) {
    T.is_open(s) and T.is_open(t) implies T.is_open(s.union(t))
} by {
    if T.is_open(s) and T.is_open(t) {
        let c: Set[T] -> Bool = pair_family(s, t)
        forall(u: Set[T]) {
            if c(u) {
                pair_family(s, t, u) = (u = s or u = t)
                c(u) = (u = s or u = t)
                u = s or u = t
                if u = s {
                    T.is_open(u)
                } else {
                    u = t
                    T.is_open(u)
                }
            }
        }
        open_big_union[T](c)
        T.is_open(big_union(c))
        big_union_pair(s, t)
        big_union(c) = s.union(t)
        T.is_open(s.union(t))
    }
}

/// The complement of a binary intersection is the union of complements.
theorem compl_intersection_eq_union_compl[T](s: Set[T], t: Set[T]) {
    s.intersection(t).c = s.c.union(t.c)
} by {
    let u = s.intersection(t)
    let v = s.c.union(t.c)
    forall(x: T) {
        intersection_contains_eq(s, t, x)
        compl_contains_eq(u, x)
        compl_contains_eq(s, x)
        compl_contains_eq(t, x)
        union_contains_eq(s.c, t.c, x)
        if u.c.contains(x) {
            not (s.contains(x) and t.contains(x))
            not s.contains(x) or not t.contains(x)
            s.c.contains(x) or t.c.contains(x)
            v.contains(x)
        }
        if v.contains(x) {
            s.c.contains(x) or t.c.contains(x)
            not s.contains(x) or not t.contains(x)
            not (s.contains(x) and t.contains(x))
            u.c.contains(x)
        }
        u.c.contains(x) = v.contains(x)
    }
    set_ext(u.c, v)
}

/// The complement of a binary union is the intersection of complements.
theorem compl_union_eq_intersection_compl[T](s: Set[T], t: Set[T]) {
    s.union(t).c = s.c.intersection(t.c)
} by {
    let u = s.union(t)
    let v = s.c.intersection(t.c)
    forall(x: T) {
        union_contains_eq(s, t, x)
        compl_contains_eq(u, x)
        compl_contains_eq(s, x)
        compl_contains_eq(t, x)
        intersection_contains_eq(s.c, t.c, x)
        if u.c.contains(x) {
            not (s.contains(x) or t.contains(x))
            (not s.contains(x)) and (not t.contains(x))
            s.c.contains(x) and t.c.contains(x)
            v.contains(x)
        }
        if v.contains(x) {
            s.c.contains(x) and t.c.contains(x)
            (not s.contains(x)) and (not t.contains(x))
            not (s.contains(x) or t.contains(x))
            u.c.contains(x)
        }
        u.c.contains(x) = v.contains(x)
    }
    set_ext(u.c, v)
}

/// The intersection of two closed sets is closed.
theorem closed_inter[T: TopologicalSpace](s: Set[T], t: Set[T]) {
    is_closed(s) and is_closed(t) implies is_closed(s.intersection(t))
} by {
    if is_closed(s) and is_closed(t) {
        T.is_open(s.c) and T.is_open(t.c)
        compl_intersection_eq_union_compl(s, t)
        s.intersection(t).c = s.c.union(t.c)
        open_union_open[T](s.c, t.c)
        T.is_open(s.c.union(t.c))
        T.is_open(s.intersection(t).c)
    }
}

/// The union of two closed sets is closed.
theorem closed_union_closed[T: TopologicalSpace](s: Set[T], t: Set[T]) {
    is_closed(s) and is_closed(t) implies is_closed(s.union(t))
} by {
    if is_closed(s) and is_closed(t) {
        T.is_open(s.c) and T.is_open(t.c)
        compl_union_eq_intersection_compl(s, t)
        s.union(t).c = s.c.intersection(t.c)
        open_inter[T](s.c, t.c)
        T.is_open(s.c.intersection(t.c))
        T.is_open(s.union(t).c)
    }
}

/// The interior of a set: the union of all open subsets.
define interior_contains[T: TopologicalSpace](s: Set[T], x: T) -> Bool {
    exists(u: Set[T]) {
        T.is_open(u) and u.subset(s) and u.contains(x)
    }
}

/// The interior of a subset is the largest open subset contained in it.
define interior[T: TopologicalSpace](s: Set[T]) -> Set[T] {
    Set[T].new(interior_contains(s))
}

/// Membership in the interior is membership in some open subset of the set.
theorem interior_contains_eq[T: TopologicalSpace](s: Set[T], x: T) {
    interior(s).contains(x) = exists(u: Set[T]) {
        T.is_open(u) and u.subset(s) and u.contains(x)
    }
} by {
    interior(s).contains(x) = interior_contains(s, x)
}

/// The interior is contained in the original set.
theorem interior_subset[T: TopologicalSpace](s: Set[T]) {
    interior(s).subset(s)
} by {
    forall(x: T) {
        if interior(s).contains(x) {
            interior_contains_eq(s, x)
            let u: Set[T] satisfy {
                T.is_open(u) and u.subset(s) and u.contains(x)
            }
            u.contains(x) implies s.contains(x)
            s.contains(x)
        }
    }
}

/// The closure of a set: the intersection of all closed supersets.
define closure_contains[T: TopologicalSpace](s: Set[T], x: T) -> Bool {
    forall(c: Set[T]) {
        is_closed(c) and s.subset(c) implies c.contains(x)
    }
}

/// The closure of a subset is the smallest closed superset.
define closure[T: TopologicalSpace](s: Set[T]) -> Set[T] {
    Set[T].new(closure_contains(s))
}

/// Membership in the closure is membership in every closed superset.
theorem closure_contains_eq[T: TopologicalSpace](s: Set[T], x: T) {
    closure(s).contains(x) = forall(c: Set[T]) {
        is_closed(c) and s.subset(c) implies c.contains(x)
    }
} by {
    closure(s).contains(x) = closure_contains(s, x)
}

/// A set is contained in its closure.
theorem subset_closure[T: TopologicalSpace](s: Set[T]) {
    s.subset(closure(s))
} by {
    forall(x: T) {
        if s.contains(x) {
            forall(c: Set[T]) {
                if is_closed(c) and s.subset(c) {
                    s.subset(c) implies (s.contains(x) implies c.contains(x))
                    c.contains(x)
                }
            }
            closure_contains_eq(s, x)
            closure(s).contains(x)
        }
    }
}

/// The frontier (boundary) of a set: closure minus interior.
define frontier[T: TopologicalSpace](s: Set[T]) -> Set[T] {
    closure(s).difference(interior(s))
}

/// True if the function `f: X -> Y` is continuous: the preimage of every open set is open.
define is_continuous[X: TopologicalSpace, Y: TopologicalSpace](f: X -> Y) -> Bool {
    forall(v: Set[Y]) {
        Y.is_open(v) implies X.is_open(set_preimage(f, v))
    }
}

/// The identity function is continuous.
theorem id_continuous[X: TopologicalSpace] {
    is_continuous(identity_fn[X])
} by {
    forall(v: Set[X]) {
        if X.is_open(v) {
            forall(y: X) {
                set_preimage_contains_eq(identity_fn[X], v, y)
                identity_fn[X](y) = y
                preimage_contains(identity_fn[X], v, y) = v.contains(y)
                set_preimage(identity_fn[X], v).contains(y) = v.contains(y)
            }
            set_ext(set_preimage(identity_fn[X], v), v)
            X.is_open(set_preimage(identity_fn[X], v))
        }
    }
}

/// The family of all open subsets of a given set.
define open_subsets_family[T: TopologicalSpace](s: Set[T], u: Set[T]) -> Bool {
    T.is_open(u) and u.subset(s)
}

/// The interior of a set equals the union of its open subsets.
theorem interior_eq_big_union_open_subsets[T: TopologicalSpace](s: Set[T]) {
    interior(s) = big_union(open_subsets_family(s))
} by {
    let f = open_subsets_family(s)
    forall(x: T) {
        if interior(s).contains(x) {
            interior_contains_eq(s, x)
            let u: Set[T] satisfy {
                T.is_open(u) and u.subset(s) and u.contains(x)
            }
            open_subsets_family(s, u) = (T.is_open(u) and u.subset(s))
            f(u) = (T.is_open(u) and u.subset(s))
            f(u)
            big_union_contains_of_member(f, u, x)
            big_union(f).contains(x)
        }
        if big_union(f).contains(x) {
            big_union_contains_eq(f, x)
            let u: Set[T] satisfy {
                f(u) and u.contains(x)
            }
            open_subsets_family(s, u) = (T.is_open(u) and u.subset(s))
            f(u) = (T.is_open(u) and u.subset(s))
            T.is_open(u) and u.subset(s)
            T.is_open(u) and u.subset(s) and u.contains(x)
            interior_contains_eq(s, x)
            interior(s).contains(x)
        }
        interior(s).contains(x) = big_union(f).contains(x)
    }
    set_ext(interior(s), big_union(f))
}

/// The interior of any set is open.
theorem interior_open[T: TopologicalSpace](s: Set[T]) {
    T.is_open(interior(s))
} by {
    let f = open_subsets_family(s)
    forall(u: Set[T]) {
        if f(u) {
            open_subsets_family(s, u) = (T.is_open(u) and u.subset(s))
            f(u) = (T.is_open(u) and u.subset(s))
            T.is_open(u)
        }
    }
    open_big_union[T](f)
    T.is_open(big_union(f))
    interior_eq_big_union_open_subsets(s)
    interior(s) = big_union(f)
}

/// The interior of a set is the largest open set contained in it: any open subset of `s` is contained in the interior.
theorem open_subset_imp_subset_interior[T: TopologicalSpace](s: Set[T], u: Set[T]) {
    T.is_open(u) and u.subset(s) implies u.subset(interior(s))
} by {
    if T.is_open(u) and u.subset(s) {
        forall(x: T) {
            if u.contains(x) {
                T.is_open(u) and u.subset(s) and u.contains(x)
                interior_contains_eq(s, x)
                interior(s).contains(x)
            }
        }
    }
}

/// An open set equals its own interior.
theorem open_eq_interior[T: TopologicalSpace](s: Set[T]) {
    T.is_open(s) implies s = interior(s)
} by {
    if T.is_open(s) {
        subset_refl(s)
        s.subset(s)
        open_subset_imp_subset_interior[T](s, s)
        s.subset(interior(s))
        interior_subset[T](s)
        interior(s).subset(s)
        subset_antisymm(s, interior(s))
    }
}

/// A set is open iff it equals its interior.
theorem open_iff_eq_interior[T: TopologicalSpace](s: Set[T]) {
    T.is_open(s) = (s = interior(s))
} by {
    if T.is_open(s) {
        open_eq_interior[T](s)
        s = interior(s)
    }
    if s = interior(s) {
        interior_open[T](s)
        T.is_open(interior(s))
        T.is_open(s)
    }
}

/// The family of all closed supersets of a given set.
define closed_supersets_family[T: TopologicalSpace](s: Set[T], u: Set[T]) -> Bool {
    is_closed(u) and s.subset(u)
}

/// The closure of a set equals the intersection of its closed supersets.
theorem closure_eq_big_intersection_closed_supersets[T: TopologicalSpace](s: Set[T]) {
    closure(s) = big_intersection(closed_supersets_family(s))
} by {
    let f = closed_supersets_family(s)
    forall(x: T) {
        if closure(s).contains(x) {
            closure_contains_eq(s, x)
            forall(u: Set[T]) {
                if f(u) {
                    closed_supersets_family(s, u) = (is_closed(u) and s.subset(u))
                    f(u) = (is_closed(u) and s.subset(u))
                    is_closed(u) and s.subset(u)
                    u.contains(x)
                }
            }
            big_intersection_contains_eq(f, x)
            big_intersection(f).contains(x)
        }
        if big_intersection(f).contains(x) {
            big_intersection_contains_eq(f, x)
            forall(u: Set[T]) {
                if is_closed(u) and s.subset(u) {
                    closed_supersets_family(s, u) = (is_closed(u) and s.subset(u))
                    f(u) = (is_closed(u) and s.subset(u))
                    f(u)
                    u.contains(x)
                }
            }
            closure_contains_eq(s, x)
            closure(s).contains(x)
        }
        closure(s).contains(x) = big_intersection(f).contains(x)
    }
    set_ext(closure(s), big_intersection(f))
}

/// Membership in the closure forces membership in every closed superset.
theorem closure_contains_imp_member[T: TopologicalSpace](s: Set[T], c: Set[T], x: T) {
    closure(s).contains(x) and is_closed(c) and s.subset(c) implies c.contains(x)
} by {
    if closure(s).contains(x) and is_closed(c) and s.subset(c) {
        closure_contains_eq(s, x)
        closure_contains(s, x)
        is_closed(c) and s.subset(c) implies c.contains(x)
        c.contains(x)
    }
}

/// The closure of a closed set is the set itself.
theorem closure_subset_of_closed[T: TopologicalSpace](s: Set[T]) {
    is_closed(s) implies closure(s).subset(s)
} by {
    if is_closed(s) {
        subset_refl(s)
        s.subset(s)
        forall(x: T) {
            if closure(s).contains(x) {
                closure_contains_imp_member[T](s, s, x)
                s.contains(x)
            }
        }
    }
}

/// A closed set is its own closure.
theorem closed_eq_closure[T: TopologicalSpace](s: Set[T]) {
    is_closed(s) implies s = closure(s)
} by {
    if is_closed(s) {
        subset_closure(s)
        s.subset(closure(s))
        closure_subset_of_closed[T](s)
        closure(s).subset(s)
        subset_antisymm(s, closure(s))
    }
}

/// The interior of the empty set is empty.
theorem interior_empty[T: TopologicalSpace] {
    interior(Set[T].empty_set) = Set[T].empty_set
} by {
    open_empty[T]
    T.is_open(Set[T].empty_set)
    open_eq_interior[T](Set[T].empty_set)
    Set[T].empty_set = interior(Set[T].empty_set)
}

/// The interior of the universal set is the universal set.
theorem interior_universal[T: TopologicalSpace] {
    interior(Set[T].universal_set) = Set[T].universal_set
} by {
    open_universal[T]
    T.is_open(Set[T].universal_set)
    open_eq_interior[T](Set[T].universal_set)
    Set[T].universal_set = interior(Set[T].universal_set)
}

/// The closure of the empty set is empty.
theorem closure_empty[T: TopologicalSpace] {
    closure(Set[T].empty_set) = Set[T].empty_set
} by {
    closed_empty[T]
    is_closed(Set[T].empty_set)
    closed_eq_closure[T](Set[T].empty_set)
    Set[T].empty_set = closure(Set[T].empty_set)
}

/// The closure of the universal set is the universal set.
theorem closure_universal[T: TopologicalSpace] {
    closure(Set[T].universal_set) = Set[T].universal_set
} by {
    closed_universal[T]
    is_closed(Set[T].universal_set)
    closed_eq_closure[T](Set[T].universal_set)
    Set[T].universal_set = closure(Set[T].universal_set)
}

/// The interior is monotone with respect to subset inclusion.
theorem interior_monotone[T: TopologicalSpace](s: Set[T], t: Set[T]) {
    s.subset(t) implies interior(s).subset(interior(t))
} by {
    if s.subset(t) {
        interior_open[T](s)
        T.is_open(interior(s))
        interior_subset[T](s)
        interior(s).subset(s)
        forall(x: T) {
            if interior(s).contains(x) {
                interior(s).subset(s) implies (interior(s).contains(x) implies s.contains(x))
                s.contains(x)
                s.subset(t) implies (s.contains(x) implies t.contains(x))
                t.contains(x)
            }
        }
        interior(s).subset(t)
        open_subset_imp_subset_interior[T](t, interior(s))
        interior(s).subset(interior(t))
    }
}

/// The closure is monotone with respect to subset inclusion.
theorem closure_monotone[T: TopologicalSpace](s: Set[T], t: Set[T]) {
    s.subset(t) implies closure(s).subset(closure(t))
} by {
    if s.subset(t) {
        forall(x: T) {
            if closure(s).contains(x) {
                forall(c: Set[T]) {
                    if is_closed(c) and t.subset(c) {
                        forall(y: T) {
                            if s.contains(y) {
                                s.subset(t) implies (s.contains(y) implies t.contains(y))
                                t.contains(y)
                                t.subset(c) implies (t.contains(y) implies c.contains(y))
                                c.contains(y)
                            }
                        }
                        s.subset(c)
                        closure_contains_imp_member[T](s, c, x)
                        c.contains(x)
                    }
                }
                closure_contains_eq(t, x)
                closure(t).contains(x)
            }
        }
    }
}

/// Interior is idempotent: applying interior twice gives the same result.
theorem interior_idempotent[T: TopologicalSpace](s: Set[T]) {
    interior(interior(s)) = interior(s)
} by {
    interior_open[T](s)
    T.is_open(interior(s))
    open_eq_interior[T](interior(s))
    interior(s) = interior(interior(s))
}

/// The complement of the closure equals the interior of the complement.
theorem compl_closure_eq_interior_compl[T: TopologicalSpace](s: Set[T]) {
    closure(s).c = interior(s.c)
} by {
    let l = closure(s).c
    let r = interior(s.c)
    forall(x: T) {
        if l.contains(x) {
            compl_contains_eq(closure(s), x)
            not closure(s).contains(x)
            closure_contains_eq(s, x)
            exists(c: Set[T]) {
                is_closed(c) and s.subset(c) and not c.contains(x)
            }
            let c: Set[T] satisfy {
                is_closed(c) and s.subset(c) and not c.contains(x)
            }
            T.is_open(c.c)
            forall(y: T) {
                if c.c.contains(y) {
                    compl_contains_eq(c, y)
                    not c.contains(y)
                    if s.contains(y) {
                        s.subset(c) implies (s.contains(y) implies c.contains(y))
                        c.contains(y)
                        false
                    }
                    not s.contains(y)
                    compl_contains_eq(s, y)
                    s.c.contains(y)
                }
            }
            c.c.subset(s.c)
            compl_contains_eq(c, x)
            c.c.contains(x)
            T.is_open(c.c) and c.c.subset(s.c) and c.c.contains(x)
            interior_contains_eq(s.c, x)
            r.contains(x)
        }
        if r.contains(x) {
            interior_contains_eq(s.c, x)
            let u: Set[T] satisfy {
                T.is_open(u) and u.subset(s.c) and u.contains(x)
            }
            is_closed(u.c)
            forall(y: T) {
                if s.contains(y) {
                    compl_contains_eq(s, y)
                    not s.c.contains(y)
                    u.subset(s.c) implies (u.contains(y) implies s.c.contains(y))
                    not u.contains(y)
                    compl_contains_eq(u, y)
                    u.c.contains(y)
                }
            }
            s.subset(u.c)
            forall(c: Set[T]) {
                if is_closed(c) and s.subset(c) and c.contains(x) {
                    true
                }
            }
            if closure(s).contains(x) {
                closure_contains_imp_member[T](s, u.c, x)
                u.c.contains(x)
                compl_contains_eq(u, x)
                not u.contains(x)
                false
            }
            not closure(s).contains(x)
            compl_contains_eq(closure(s), x)
            l.contains(x)
        }
        l.contains(x) = r.contains(x)
    }
    set_ext(l, r)
}

/// The closure is closed.
theorem closure_closed[T: TopologicalSpace](s: Set[T]) {
    is_closed(closure(s))
} by {
    compl_closure_eq_interior_compl[T](s)
    closure(s).c = interior(s.c)
    interior_open[T](s.c)
    T.is_open(interior(s.c))
    T.is_open(closure(s).c)
}

/// Closure is idempotent: applying closure twice gives the same result.
theorem closure_idempotent[T: TopologicalSpace](s: Set[T]) {
    closure(closure(s)) = closure(s)
} by {
    closure_closed[T](s)
    is_closed(closure(s))
    closed_eq_closure[T](closure(s))
    closure(s) = closure(closure(s))
}

/// The closure equals the complement of the interior of the complement.
theorem closure_eq_compl_interior_compl[T: TopologicalSpace](s: Set[T]) {
    closure(s) = interior(s.c).c
} by {
    compl_closure_eq_interior_compl[T](s)
    closure(s).c = interior(s.c)
    let lhs = closure(s)
    let rhs = interior(s.c).c
    forall(x: T) {
        compl_contains_eq(closure(s), x)
        compl_contains_eq(interior(s.c), x)
        if lhs.contains(x) {
            closure(s).contains(x)
            not closure(s).c.contains(x)
            not interior(s.c).contains(x)
            interior(s.c).c.contains(x)
            rhs.contains(x)
        }
        if rhs.contains(x) {
            not interior(s.c).contains(x)
            not closure(s).c.contains(x)
            closure(s).contains(x)
            lhs.contains(x)
        }
        lhs.contains(x) = rhs.contains(x)
    }
    set_ext(lhs, rhs)
}

/// The interior equals the complement of the closure of the complement.
theorem interior_eq_compl_closure_compl[T: TopologicalSpace](s: Set[T]) {
    interior(s) = closure(s.c).c
} by {
    compl_closure_eq_interior_compl[T](s.c)
    closure(s.c).c = interior(s.c.c)
    let lhs = interior(s)
    let rhs = closure(s.c).c
    forall(x: T) {
        compl_contains_eq(s, x)
        compl_contains_eq(s.c, x)
        if lhs.contains(x) {
            interior_contains_eq(s, x)
            let u: Set[T] satisfy {
                T.is_open(u) and u.subset(s) and u.contains(x)
            }
            forall(y: T) {
                if u.contains(y) {
                    u.subset(s) implies (u.contains(y) implies s.contains(y))
                    s.contains(y)
                    compl_contains_eq(s.c, y)
                    not s.c.contains(y)
                    s.c.c.contains(y)
                }
            }
            u.subset(s.c.c)
            T.is_open(u) and u.subset(s.c.c) and u.contains(x)
            interior_contains_eq(s.c.c, x)
            interior(s.c.c).contains(x)
            closure(s.c).c.contains(x)
            rhs.contains(x)
        }
        if rhs.contains(x) {
            closure(s.c).c.contains(x)
            interior(s.c.c).contains(x)
            interior_contains_eq(s.c.c, x)
            let u: Set[T] satisfy {
                T.is_open(u) and u.subset(s.c.c) and u.contains(x)
            }
            forall(y: T) {
                if u.contains(y) {
                    u.subset(s.c.c) implies (u.contains(y) implies s.c.c.contains(y))
                    s.c.c.contains(y)
                    compl_contains_eq(s.c, y)
                    not s.c.contains(y)
                    compl_contains_eq(s, y)
                    s.contains(y)
                }
            }
            u.subset(s)
            T.is_open(u) and u.subset(s) and u.contains(x)
            interior_contains_eq(s, x)
            interior(s).contains(x)
            lhs.contains(x)
        }
        lhs.contains(x) = rhs.contains(x)
    }
    set_ext(lhs, rhs)
}

/// The frontier of a set is closed.
theorem frontier_closed[T: TopologicalSpace](s: Set[T]) {
    is_closed(frontier(s))
} by {
    closure_closed[T](s)
    is_closed(closure(s))
    T.is_open(closure(s).c)
    interior_open[T](s)
    T.is_open(interior(s))
    open_union_open[T](closure(s).c, interior(s))
    T.is_open(closure(s).c.union(interior(s)))
    let f = frontier(s)
    let g = closure(s).c.union(interior(s))
    forall(x: T) {
        compl_contains_eq(f, x)
        union_contains_eq(closure(s).c, interior(s), x)
        compl_contains_eq(closure(s), x)
        difference_contains_eq(closure(s), interior(s), x)
        f.contains(x) = (closure(s).contains(x) and not interior(s).contains(x))
        if f.c.contains(x) {
            not f.contains(x)
            not (closure(s).contains(x) and not interior(s).contains(x))
            not closure(s).contains(x) or interior(s).contains(x)
            closure(s).c.contains(x) or interior(s).contains(x)
            g.contains(x)
        }
        if g.contains(x) {
            closure(s).c.contains(x) or interior(s).contains(x)
            not closure(s).contains(x) or interior(s).contains(x)
            not (closure(s).contains(x) and not interior(s).contains(x))
            not f.contains(x)
            f.c.contains(x)
        }
        f.c.contains(x) = g.contains(x)
    }
    set_ext(f.c, g)
    f.c = g
    T.is_open(f.c)
}

/// The interior distributes over binary intersection.
theorem interior_inter[T: TopologicalSpace](s: Set[T], t: Set[T]) {
    interior(s.intersection(t)) = interior(s).intersection(interior(t))
} by {
    let lhs = interior(s.intersection(t))
    let rhs = interior(s).intersection(interior(t))
    forall(x: T) {
        intersection_contains_eq(interior(s), interior(t), x)
        intersection_contains_eq(s, t, x)
        if lhs.contains(x) {
            interior_contains_eq(s.intersection(t), x)
            let u: Set[T] satisfy {
                T.is_open(u) and u.subset(s.intersection(t)) and u.contains(x)
            }
            forall(y: T) {
                if u.contains(y) {
                    u.subset(s.intersection(t)) implies (u.contains(y) implies s.intersection(t).contains(y))
                    s.intersection(t).contains(y)
                    intersection_contains_eq(s, t, y)
                    s.contains(y) and t.contains(y)
                }
            }
            forall(y: T) {
                if u.contains(y) {
                    s.contains(y) and t.contains(y)
                    s.contains(y)
                }
            }
            u.subset(s)
            forall(y: T) {
                if u.contains(y) {
                    s.contains(y) and t.contains(y)
                    t.contains(y)
                }
            }
            u.subset(t)
            T.is_open(u) and u.subset(s) and u.contains(x)
            interior_contains_eq(s, x)
            interior(s).contains(x)
            T.is_open(u) and u.subset(t) and u.contains(x)
            interior_contains_eq(t, x)
            interior(t).contains(x)
            rhs.contains(x)
        }
        if rhs.contains(x) {
            interior(s).contains(x) and interior(t).contains(x)
            interior_contains_eq(s, x)
            let u: Set[T] satisfy {
                T.is_open(u) and u.subset(s) and u.contains(x)
            }
            interior_contains_eq(t, x)
            let v: Set[T] satisfy {
                T.is_open(v) and v.subset(t) and v.contains(x)
            }
            open_inter[T](u, v)
            T.is_open(u.intersection(v))
            forall(y: T) {
                if u.intersection(v).contains(y) {
                    intersection_contains_eq(u, v, y)
                    u.contains(y) and v.contains(y)
                    u.subset(s) implies (u.contains(y) implies s.contains(y))
                    s.contains(y)
                    v.subset(t) implies (v.contains(y) implies t.contains(y))
                    t.contains(y)
                    intersection_contains_eq(s, t, y)
                    s.intersection(t).contains(y)
                }
            }
            u.intersection(v).subset(s.intersection(t))
            intersection_contains_eq(u, v, x)
            u.intersection(v).contains(x)
            T.is_open(u.intersection(v)) and u.intersection(v).subset(s.intersection(t)) and u.intersection(v).contains(x)
            interior_contains_eq(s.intersection(t), x)
            lhs.contains(x)
        }
        lhs.contains(x) = rhs.contains(x)
    }
    set_ext(lhs, rhs)
}

/// The closure distributes over binary union.
theorem closure_union[T: TopologicalSpace](s: Set[T], t: Set[T]) {
    closure(s.union(t)) = closure(s).union(closure(t))
} by {
    let lhs = closure(s.union(t))
    let rhs = closure(s).union(closure(t))
    closure_closed[T](s)
    closure_closed[T](t)
    closed_union_closed[T](closure(s), closure(t))
    is_closed(rhs)
    forall(y: T) {
        if s.union(t).contains(y) {
            union_contains_eq(s, t, y)
            if s.contains(y) {
                subset_closure(s)
                s.subset(closure(s)) implies (s.contains(y) implies closure(s).contains(y))
                closure(s).contains(y)
                union_contains_eq(closure(s), closure(t), y)
                rhs.contains(y)
            } else {
                t.contains(y)
                subset_closure(t)
                t.subset(closure(t)) implies (t.contains(y) implies closure(t).contains(y))
                closure(t).contains(y)
                union_contains_eq(closure(s), closure(t), y)
                rhs.contains(y)
            }
        }
    }
    s.union(t).subset(rhs)
    forall(x: T) {
        if lhs.contains(x) {
            closure_contains_imp_member[T](s.union(t), rhs, x)
            rhs.contains(x)
        }
        if rhs.contains(x) {
            union_contains_eq(closure(s), closure(t), x)
            if closure(s).contains(x) {
                forall(y: T) {
                    if s.contains(y) {
                        union_contains_eq(s, t, y)
                        s.union(t).contains(y)
                    }
                }
                s.subset(s.union(t))
                closure_monotone[T](s, s.union(t))
                closure(s).subset(closure(s.union(t)))
                closure(s).subset(closure(s.union(t))) implies (closure(s).contains(x) implies closure(s.union(t)).contains(x))
                lhs.contains(x)
            } else {
                closure(t).contains(x)
                forall(y: T) {
                    if t.contains(y) {
                        union_contains_eq(s, t, y)
                        s.union(t).contains(y)
                    }
                }
                t.subset(s.union(t))
                closure_monotone[T](t, s.union(t))
                closure(t).subset(closure(s.union(t)))
                closure(t).subset(closure(s.union(t))) implies (closure(t).contains(x) implies closure(s.union(t)).contains(x))
                lhs.contains(x)
            }
        }
        lhs.contains(x) = rhs.contains(x)
    }
    set_ext(lhs, rhs)
}

/// The closure of an intersection is contained in the intersection of closures.
theorem closure_inter_subset[T: TopologicalSpace](s: Set[T], t: Set[T]) {
    closure(s.intersection(t)).subset(closure(s).intersection(closure(t)))
} by {
    sets_subset_intersection(s, t)
    s.intersection(t).subset(s)
    s.intersection(t).subset(t)
    closure_monotone[T](s.intersection(t), s)
    closure(s.intersection(t)).subset(closure(s))
    closure_monotone[T](s.intersection(t), t)
    closure(s.intersection(t)).subset(closure(t))
    set_subset_inf_of_subset_left_right[T](closure(s.intersection(t)), closure(s), closure(t))
    closure(s.intersection(t)).subset(set_inf(closure(s), closure(t)))
    set_inf(closure(s), closure(t)) = closure(s).intersection(closure(t))
}

/// The union of interiors is contained in the interior of the union.
theorem interior_union_supset[T: TopologicalSpace](s: Set[T], t: Set[T]) {
    interior(s).union(interior(t)).subset(interior(s.union(t)))
} by {
    sets_subset_union(s, t)
    s.subset(s.union(t))
    t.subset(s.union(t))
    interior_monotone[T](s, s.union(t))
    interior(s).subset(interior(s.union(t)))
    interior_monotone[T](t, s.union(t))
    interior(t).subset(interior(s.union(t)))
    sets_subset_contain_union[T](interior(s), interior(t), interior(s.union(t)))
    interior(s).union(interior(t)).subset(interior(s.union(t)))
}

/// The interior of a set is contained in its closure.
theorem interior_subset_closure[T: TopologicalSpace](s: Set[T]) {
    interior(s).subset(closure(s))
} by {
    interior_subset[T](s)
    interior(s).subset(s)
    subset_closure[T](s)
    s.subset(closure(s))
    subset_trans(interior(s), s, closure(s))
}

/// A closed set containing `s` also contains the closure of `s`.
theorem closed_superset_contains_closure[T: TopologicalSpace](s: Set[T], t: Set[T]) {
    is_closed[T](t) and s.subset(t) implies closure(s).subset(t)
} by {
    if is_closed[T](t) and s.subset(t) {
        closure_monotone[T](s, t)
        closure(s).subset(closure(t))
        closed_eq_closure[T](t)
        t = closure(t)
        closure(t) = t
        closure(s).subset(t)
    }
}

/// An open set contained in `s` is also contained in the interior of `s`.
theorem open_subset_contained_in_interior[T: TopologicalSpace](u: Set[T], s: Set[T]) {
    T.is_open(u) and u.subset(s) implies u.subset(interior(s))
} by {
    if T.is_open(u) and u.subset(s) {
        open_subset_imp_subset_interior[T](s, u)
    }
}

/// The frontier of a set equals the intersection of its closure and the closure of its complement.
theorem frontier_eq_closure_inter_closure_compl[T: TopologicalSpace](s: Set[T]) {
    frontier(s) = closure(s).intersection(closure(s.c))
} by {
    let lhs = frontier(s)
    let rhs = closure(s).intersection(closure(s.c))
    interior_eq_compl_closure_compl[T](s)
    interior(s) = closure(s.c).c
    forall(x: T) {
        difference_contains_eq(closure(s), interior(s), x)
        intersection_contains_eq(closure(s), closure(s.c), x)
        compl_contains_eq(closure(s.c), x)
        if lhs.contains(x) {
            closure(s).contains(x) and not interior(s).contains(x)
            not closure(s.c).c.contains(x)
            closure(s.c).contains(x)
            rhs.contains(x)
        }
        if rhs.contains(x) {
            closure(s).contains(x) and closure(s.c).contains(x)
            not closure(s.c).c.contains(x)
            not interior(s).contains(x)
            lhs.contains(x)
        }
        lhs.contains(x) = rhs.contains(x)
    }
    set_ext(lhs, rhs)
}

/// The frontier of a set equals the frontier of its complement.
theorem frontier_compl[T: TopologicalSpace](s: Set[T]) {
    frontier(s.c) = frontier(s)
} by {
    frontier_eq_closure_inter_closure_compl[T](s)
    frontier(s) = closure(s).intersection(closure(s.c))
    frontier_eq_closure_inter_closure_compl[T](s.c)
    frontier(s.c) = closure(s.c).intersection(closure(s.c.c))
    compl_of_compl_is_self(s)
    s.c.c = s
    closure(s.c.c) = closure(s)
    closure(s.c).intersection(closure(s.c.c)) = closure(s.c).intersection(closure(s))
    forall(x: T) {
        intersection_contains_eq(closure(s.c), closure(s), x)
        intersection_contains_eq(closure(s), closure(s.c), x)
        closure(s.c).intersection(closure(s)).contains(x) = closure(s).intersection(closure(s.c)).contains(x)
    }
    set_ext(closure(s.c).intersection(closure(s)), closure(s).intersection(closure(s.c)))
    closure(s.c).intersection(closure(s)) = closure(s).intersection(closure(s.c))
    frontier(s.c) = closure(s).intersection(closure(s.c))
    frontier(s.c) = frontier(s)
}

/// A function is continuous iff the preimage of every open set is open.
theorem continuous_iff_preimage_open[X: TopologicalSpace, Y: TopologicalSpace](f: X -> Y) {
    is_continuous(f) = forall(v: Set[Y]) {
        Y.is_open(v) implies X.is_open(set_preimage(f, v))
    }
} by {
    is_continuous(f) = forall(v: Set[Y]) {
        Y.is_open(v) implies X.is_open(set_preimage(f, v))
    }
}

/// The composition of continuous functions is continuous.
theorem continuous_compose[X: TopologicalSpace, Y: TopologicalSpace, Z: TopologicalSpace](
    f: X -> Y, g: Y -> Z) {
    is_continuous(f) and is_continuous(g) implies is_continuous(compose(g, f))
} by {
    if is_continuous(f) and is_continuous(g) {
        is_continuous(f) = forall(v: Set[Y]) {
            Y.is_open(v) implies X.is_open(set_preimage(f, v))
        }
        is_continuous(g) = forall(v: Set[Z]) {
            Z.is_open(v) implies Y.is_open(set_preimage(g, v))
        }
        forall(w: Set[Z]) {
            if Z.is_open(w) {
                Y.is_open(set_preimage(g, w))
                X.is_open(set_preimage(f, set_preimage(g, w)))
                forall(x: X) {
                    set_preimage_contains_eq(compose(g, f), w, x)
                    compose(g, f, x) = g(f(x))
                    preimage_contains(compose(g, f), w, x) = w.contains(g(f(x)))
                    set_preimage(compose(g, f), w).contains(x) = w.contains(g(f(x)))
                    set_preimage_contains_eq(f, set_preimage(g, w), x)
                    preimage_contains(f, set_preimage(g, w), x) = set_preimage(g, w).contains(f(x))
                    set_preimage(f, set_preimage(g, w)).contains(x) = set_preimage(g, w).contains(f(x))
                    set_preimage_contains_eq(g, w, f(x))
                    preimage_contains(g, w, f(x)) = w.contains(g(f(x)))
                    set_preimage(g, w).contains(f(x)) = w.contains(g(f(x)))
                    set_preimage(compose(g, f), w).contains(x) = set_preimage(f, set_preimage(g, w)).contains(x)
                }
                set_ext(set_preimage(compose(g, f), w), set_preimage(f, set_preimage(g, w)))
                X.is_open(set_preimage(compose(g, f), w))
            }
        }
    }
}

/// True if a set has a nonempty intersection with another set.
define intersects[T](s: Set[T], t: Set[T]) -> Bool {
    exists(x: T) {
        s.contains(x) and t.contains(x)
    }
}

/// True if `(f, g)` is a homeomorphism pair: `g` is the inverse of `f`, and both `f` and
/// `g` are continuous.
define is_homeomorphism[X: TopologicalSpace, Y: TopologicalSpace](f: X -> Y, g: Y -> X) -> Bool {
    is_continuous(f) and is_continuous(g)
        and (forall(x: X) { g(f(x)) = x })
        and (forall(y: Y) { f(g(y)) = y })
}

/// `(identity, identity)` is a homeomorphism on any topological space.
theorem identity_is_homeomorphism[X: TopologicalSpace] {
    is_homeomorphism[X, X](identity_fn[X], identity_fn[X])
} by {
    id_continuous[X]
    is_continuous(identity_fn[X])
    forall(x: X) {
        identity_fn[X](identity_fn[X](x)) = x
    }
}

/// Unfolding lemma for homeomorphism.
theorem is_homeomorphism_iff[X: TopologicalSpace, Y: TopologicalSpace](
    f: X -> Y, g: Y -> X) {
    is_homeomorphism[X, Y](f, g) = (is_continuous(f) and is_continuous(g)
        and (forall(x: X) { g(f(x)) = x })
        and (forall(y: Y) { f(g(y)) = y }))
}

/// The forward map of a homeomorphism is continuous.
theorem is_homeomorphism_continuous_fwd[X: TopologicalSpace, Y: TopologicalSpace](
    f: X -> Y, g: Y -> X) {
    is_homeomorphism[X, Y](f, g) implies is_continuous(f)
} by {
    if is_homeomorphism[X, Y](f, g) {
        is_homeomorphism_iff[X, Y](f, g)
    }
}

/// The inverse map of a homeomorphism is continuous.
theorem is_homeomorphism_continuous_inv[X: TopologicalSpace, Y: TopologicalSpace](
    f: X -> Y, g: Y -> X) {
    is_homeomorphism[X, Y](f, g) implies is_continuous(g)
} by {
    if is_homeomorphism[X, Y](f, g) {
        is_homeomorphism_iff[X, Y](f, g)
    }
}

/// The inverse map is a left inverse of the forward map of a homeomorphism.
theorem is_homeomorphism_left_inv[X: TopologicalSpace, Y: TopologicalSpace](
    f: X -> Y, g: Y -> X, x: X) {
    is_homeomorphism[X, Y](f, g) implies g(f(x)) = x
} by {
    if is_homeomorphism[X, Y](f, g) {
        is_homeomorphism_iff[X, Y](f, g)
    }
}

/// The inverse map is a right inverse of the forward map of a homeomorphism.
theorem is_homeomorphism_right_inv[X: TopologicalSpace, Y: TopologicalSpace](
    f: X -> Y, g: Y -> X, y: Y) {
    is_homeomorphism[X, Y](f, g) implies f(g(y)) = y
} by {
    if is_homeomorphism[X, Y](f, g) {
        is_homeomorphism_iff[X, Y](f, g)
    }
}

/// The pointwise left-inverse identity for the composed maps of two
/// homeomorphism pairs.
theorem homeomorphism_compose_left_inv[X: TopologicalSpace, Y: TopologicalSpace, Z: TopologicalSpace](
    f1: X -> Y, g1: Y -> X, f2: Y -> Z, g2: Z -> Y, x: X) {
    is_homeomorphism[X, Y](f1, g1) and is_homeomorphism[Y, Z](f2, g2)
        implies compose(g1, g2, compose(f2, f1, x)) = x
} by {
    if is_homeomorphism[X, Y](f1, g1) and is_homeomorphism[Y, Z](f2, g2) {
        is_homeomorphism_left_inv[X, Y](f1, g1, x)
        is_homeomorphism_left_inv[Y, Z](f2, g2, f1(x))
        g1(f1(x)) = x
        g2(f2(f1(x))) = f1(x)
        compose(f2, f1, x) = f2(f1(x))
        compose(g1, g2, f2(f1(x))) = g1(g2(f2(f1(x))))
        compose(g1, g2, compose(f2, f1, x)) = g1(g2(f2(f1(x))))
        compose(g1, g2, compose(f2, f1, x)) = g1(f1(x))
        compose(g1, g2, compose(f2, f1, x)) = x
    }
}

/// The pointwise right-inverse identity for the composed maps of two
/// homeomorphism pairs.
theorem homeomorphism_compose_right_inv[X: TopologicalSpace, Y: TopologicalSpace, Z: TopologicalSpace](
    f1: X -> Y, g1: Y -> X, f2: Y -> Z, g2: Z -> Y, z: Z) {
    is_homeomorphism[X, Y](f1, g1) and is_homeomorphism[Y, Z](f2, g2)
        implies compose(f2, f1, compose(g1, g2, z)) = z
} by {
    if is_homeomorphism[X, Y](f1, g1) and is_homeomorphism[Y, Z](f2, g2) {
        is_homeomorphism_right_inv[Y, Z](f2, g2, z)
        is_homeomorphism_right_inv[X, Y](f1, g1, g2(z))
        f2(g2(z)) = z
        f1(g1(g2(z))) = g2(z)
        compose(g1, g2, z) = g1(g2(z))
        compose(f2, f1, g1(g2(z))) = f2(f1(g1(g2(z))))
        compose(f2, f1, compose(g1, g2, z)) = f2(f1(g1(g2(z))))
        compose(f2, f1, compose(g1, g2, z)) = f2(g2(z))
        compose(f2, f1, compose(g1, g2, z)) = z
    }
}

/// True if there is a homeomorphism between two topological spaces.
define is_homeomorphic[X: TopologicalSpace, Y: TopologicalSpace](
    witness_x: X, witness_y: Y) -> Bool {
    exists(f: X -> Y, g: Y -> X) {
        is_homeomorphism[X, Y](f, g)
    }
}


/// True if the openness predicate makes the space a T0 space: any two distinct points
/// are topologically distinguishable by some open set.
define is_t0_space[T](open: Set[T] -> Bool) -> Bool {
    forall(x: T, y: T) {
        x != y implies exists(u: Set[T]) {
            open(u) and ((u.contains(x) and not u.contains(y))
                or (u.contains(y) and not u.contains(x)))
        }
    }
}

/// True if the openness predicate makes the space a T1 space: for any two distinct
/// points, each has an open neighborhood missing the other.
define is_t1_space[T](open: Set[T] -> Bool) -> Bool {
    forall(x: T, y: T) {
        x != y implies exists(u: Set[T]) {
            open(u) and u.contains(x) and not u.contains(y)
        }
    }
}

/// True if the openness predicate makes the space Hausdorff (T2): any two distinct
/// points have disjoint open neighborhoods.
define is_hausdorff[T](open: Set[T] -> Bool) -> Bool {
    forall(x: T, y: T) {
        x != y implies exists(u: Set[T], v: Set[T]) {
            open(u) and open(v) and u.contains(x) and v.contains(y) and u.is_disjoint(v)
        }
    }
}

/// Unfolding lemma for `is_t0_space`.
theorem is_t0_space_iff[T](open: Set[T] -> Bool) {
    is_t0_space[T](open) = forall(x: T, y: T) {
        x != y implies exists(u: Set[T]) {
            open(u) and ((u.contains(x) and not u.contains(y))
                or (u.contains(y) and not u.contains(x)))
        }
    }
}

/// Unfolding lemma for `is_t1_space`.
theorem is_t1_space_iff[T](open: Set[T] -> Bool) {
    is_t1_space[T](open) = forall(x: T, y: T) {
        x != y implies exists(u: Set[T]) {
            open(u) and u.contains(x) and not u.contains(y)
        }
    }
}

/// Unfolding lemma for `is_hausdorff`.
theorem is_hausdorff_iff[T](open: Set[T] -> Bool) {
    is_hausdorff[T](open) = forall(x: T, y: T) {
        x != y implies exists(u: Set[T], v: Set[T]) {
            open(u) and open(v) and u.contains(x) and v.contains(y) and u.is_disjoint(v)
        }
    }
}

/// Two disjoint sets share no point.
theorem disjoint_not_both[K](u: Set[K], v: Set[K], y: K) {
    u.is_disjoint(v) implies not (u.contains(y) and v.contains(y))
} by {
    if u.is_disjoint(v) {
        u.is_disjoint(v) = forall(z: K) { not (u.contains(z) and v.contains(z)) }
        forall(z: K) { not (u.contains(z) and v.contains(z)) }
        not (u.contains(y) and v.contains(y))
    }
}

/// Instantiation of the T1 condition at a fixed pair of distinct points.
theorem t1_space_inst[T](open: Set[T] -> Bool, x: T, y: T) {
    is_t1_space[T](open) and x != y implies exists(u: Set[T]) {
        open(u) and u.contains(x) and not u.contains(y)
    }
} by {
    if is_t1_space[T](open) and x != y {
        is_t1_space_iff[T](open)
        x != y implies exists(u: Set[T]) {
            open(u) and u.contains(x) and not u.contains(y)
        }
    }
}

/// Instantiation of the Hausdorff condition at a fixed pair of distinct points.
theorem hausdorff_inst[T](open: Set[T] -> Bool, x: T, y: T) {
    is_hausdorff[T](open) and x != y implies exists(u: Set[T], v: Set[T]) {
        open(u) and open(v) and u.contains(x) and v.contains(y) and u.is_disjoint(v)
    }
} by {
    if is_hausdorff[T](open) and x != y {
        is_hausdorff_iff[T](open)
        x != y implies exists(u: Set[T], v: Set[T]) {
            open(u) and open(v) and u.contains(x) and v.contains(y)
                and u.is_disjoint(v)
        }
    }
}

/// A neighborhood of `x` missing `y` distinguishes `x` from `y` in the T0 sense.
theorem t0_witness_of_t1_witness[T](open: Set[T] -> Bool, x: T, y: T, u: Set[T]) {
    open(u) and u.contains(x) and not u.contains(y) implies exists(w: Set[T]) {
        open(w) and ((w.contains(x) and not w.contains(y))
            or (w.contains(y) and not w.contains(x)))
    }
} by {
    if open(u) and u.contains(x) and not u.contains(y) {
        open(u) and ((u.contains(x) and not u.contains(y))
            or (u.contains(y) and not u.contains(x)))
    }
}

/// Disjoint open neighborhoods of `x` and `y` give an open neighborhood of `x`
/// that misses `y`.
theorem t1_witness_of_hausdorff_witness[T](
    open: Set[T] -> Bool, x: T, y: T, u: Set[T], v: Set[T]
) {
    open(u) and u.contains(x) and v.contains(y) and u.is_disjoint(v)
        implies exists(w: Set[T]) {
            open(w) and w.contains(x) and not w.contains(y)
        }
} by {
    if open(u) and u.contains(x) and v.contains(y) and u.is_disjoint(v) {
        disjoint_not_both[T](u, v, y)
        not (u.contains(y) and v.contains(y))
        not u.contains(y)
        open(u) and u.contains(x) and not u.contains(y)
    }
}

/// Every T1 space is a T0 space: a neighborhood of `x` missing `y` already
/// distinguishes the two points.
theorem t1_imp_t0[T](open: Set[T] -> Bool) {
    is_t1_space[T](open) implies is_t0_space[T](open)
} by {
    if is_t1_space[T](open) {
        is_t1_space_iff[T](open)
        is_t0_space_iff[T](open)
        forall(x: T, y: T) {
            if x != y {
                t1_space_inst[T](open, x, y)
                let u: Set[T] satisfy {
                    open(u) and u.contains(x) and not u.contains(y)
                }
                t0_witness_of_t1_witness[T](open, x, y, u)
                exists(w: Set[T]) {
                    open(w) and ((w.contains(x) and not w.contains(y))
                        or (w.contains(y) and not w.contains(x)))
                }
            }
        }
    }
}

/// Every Hausdorff space is a T1 space: one of the disjoint open neighborhoods
/// separating `x` from `y` is a neighborhood of `x` missing `y`.
theorem hausdorff_imp_t1[T](open: Set[T] -> Bool) {
    is_hausdorff[T](open) implies is_t1_space[T](open)
} by {
    if is_hausdorff[T](open) {
        is_hausdorff_iff[T](open)
        is_t1_space_iff[T](open)
        forall(x: T, y: T) {
            if x != y {
                hausdorff_inst[T](open, x, y)
                let u: Set[T] satisfy {
                    exists(vv: Set[T]) {
                        open(u) and open(vv) and u.contains(x) and vv.contains(y)
                            and u.is_disjoint(vv)
                    }
                }
                let v: Set[T] satisfy {
                    open(u) and open(v) and u.contains(x) and v.contains(y)
                        and u.is_disjoint(v)
                }
                t1_witness_of_hausdorff_witness[T](open, x, y, u, v)
                exists(w: Set[T]) {
                    open(w) and w.contains(x) and not w.contains(y)
                }
            }
        }
    }
}

/// Every Hausdorff space is a T0 space.
theorem hausdorff_imp_t0[T](open: Set[T] -> Bool) {
    is_hausdorff[T](open) implies is_t0_space[T](open)
} by {
    if is_hausdorff[T](open) {
        hausdorff_imp_t1[T](open)
        t1_imp_t0[T](open)
    }
}

/// True if a subset is clopen: both itself and its complement are open.
define is_clopen[T](open: Set[T] -> Bool, s: Set[T]) -> Bool {
    open(s) and open(s.c)
}

/// Unfolding lemma for `is_clopen`.
theorem is_clopen_iff[T](open: Set[T] -> Bool, s: Set[T]) {
    is_clopen[T](open, s) = (open(s) and open(s.c))
}

/// The empty set is clopen in any topology that contains the universal set as open.
theorem clopen_empty[T: TopologicalSpace] {
    is_clopen[T](T.is_open, Set[T].empty_set)
} by {
    open_empty[T]
    empty_set_compl_is_universal[T]
    Set[T].empty_set.c = Set[T].universal_set
    open_universal[T]
    T.is_open(Set[T].universal_set)
    T.is_open(Set[T].empty_set.c)
}

/// The universal set is clopen in any topology.
theorem clopen_universal[T: TopologicalSpace] {
    is_clopen[T](T.is_open, Set[T].universal_set)
} by {
    open_universal[T]
    universal_set_compl_is_empty[T]
    Set[T].universal_set.c = Set[T].empty_set
    open_empty[T]
    T.is_open(Set[T].empty_set)
    T.is_open(Set[T].universal_set.c)
}

/// Builder for is_clopen from open and complement-open assumptions.
theorem is_clopen_intro[T](open: Set[T] -> Bool, s: Set[T]) {
    open(s) and open(s.c) implies is_clopen[T](open, s)
}

/// Extract opennesses from a clopen.
theorem is_clopen_imp_open[T](open: Set[T] -> Bool, s: Set[T]) {
    is_clopen[T](open, s) implies open(s) and open(s.c)
}

/// The complement of a clopen set is clopen.
theorem is_clopen_compl[T: TopologicalSpace](s: Set[T]) {
    is_clopen[T](T.is_open, s) implies is_clopen[T](T.is_open, s.c)
} by {
    if is_clopen[T](T.is_open, s) {
        is_clopen_imp_open[T](T.is_open, s)
        T.is_open(s) and T.is_open(s.c)
        compl_of_compl_is_self(s)
        s.c.c = s
        T.is_open(s.c.c)
        T.is_open(s.c) and T.is_open(s.c.c)
        is_clopen_intro[T](T.is_open, s.c)
    }
}

/// The intersection of two clopen sets is clopen.
theorem is_clopen_inter[T: TopologicalSpace](s: Set[T], t: Set[T]) {
    is_clopen[T](T.is_open, s) and is_clopen[T](T.is_open, t)
        implies is_clopen[T](T.is_open, s.intersection(t))
} by {
    if is_clopen[T](T.is_open, s) and is_clopen[T](T.is_open, t) {
        is_clopen_imp_open[T](T.is_open, s)
        is_clopen_imp_open[T](T.is_open, t)
        T.is_open(s) and T.is_open(s.c)
        T.is_open(t) and T.is_open(t.c)
        open_inter[T](s, t)
        T.is_open(s.intersection(t))
        open_union_open[T](s.c, t.c)
        T.is_open(s.c.union(t.c))
        compl_intersection_eq_union_compl(s, t)
        s.intersection(t).c = s.c.union(t.c)
        T.is_open(s.intersection(t).c)
        T.is_open(s.intersection(t)) and T.is_open(s.intersection(t).c)
        is_clopen_intro[T](T.is_open, s.intersection(t))
        is_clopen[T](T.is_open, s.intersection(t))
    }
}

/// The union of two clopen sets is clopen.
theorem is_clopen_union[T: TopologicalSpace](s: Set[T], t: Set[T]) {
    is_clopen[T](T.is_open, s) and is_clopen[T](T.is_open, t)
        implies is_clopen[T](T.is_open, s.union(t))
} by {
    if is_clopen[T](T.is_open, s) and is_clopen[T](T.is_open, t) {
        is_clopen_imp_open[T](T.is_open, s)
        is_clopen_imp_open[T](T.is_open, t)
        T.is_open(s) and T.is_open(s.c)
        T.is_open(t) and T.is_open(t.c)
        open_union_open[T](s, t)
        T.is_open(s.union(t))
        open_inter[T](s.c, t.c)
        T.is_open(s.c.intersection(t.c))
        compl_union_eq_intersection_compl(s, t)
        s.union(t).c = s.c.intersection(t.c)
        T.is_open(s.union(t).c)
        T.is_open(s.union(t)) and T.is_open(s.union(t).c)
        is_clopen_intro[T](T.is_open, s.union(t))
        is_clopen[T](T.is_open, s.union(t))
    }
}

/// True if `(u, v)` is a disconnection witness for the openness `open`: two disjoint
/// nonempty open sets that cover the whole space.
define is_disconnection[T](open: Set[T] -> Bool, u: Set[T], v: Set[T]) -> Bool {
    open(u) and open(v) and u != Set[T].empty_set and v != Set[T].empty_set
        and u.is_disjoint(v) and u.union(v) = Set[T].universal_set
}

/// Unfolding lemma for `is_disconnection`.
theorem is_disconnection_iff[T](open: Set[T] -> Bool, u: Set[T], v: Set[T]) {
    is_disconnection[T](open, u, v) = (open(u) and open(v)
        and u != Set[T].empty_set and v != Set[T].empty_set
        and u.is_disjoint(v) and u.union(v) = Set[T].universal_set)
}

/// Builder lemma for `is_disconnection` from the component pieces.
theorem is_disconnection_intro[T](open: Set[T] -> Bool, u: Set[T], v: Set[T]) {
    open(u) and open(v) and u != Set[T].empty_set and v != Set[T].empty_set
        and u.is_disjoint(v) and u.union(v) = Set[T].universal_set
        implies is_disconnection[T](open, u, v)
}

/// True if a topological space (described by an openness predicate) is connected:
/// it admits no disconnection.
define is_connected[T](open: Set[T] -> Bool) -> Bool {
    not exists(u: Set[T], v: Set[T]) {
        is_disconnection[T](open, u, v)
    }
}

/// Unfolding lemma for `is_connected`.
theorem is_connected_iff[T](open: Set[T] -> Bool) {
    is_connected[T](open) = not exists(u: Set[T], v: Set[T]) {
        is_disconnection[T](open, u, v)
    }
}

/// True if the only clopen sets are the empty set and the universal set.
define is_connected_via_clopen[T](open: Set[T] -> Bool) -> Bool {
    forall(s: Set[T]) {
        is_clopen[T](open, s) implies (s = Set[T].empty_set or s = Set[T].universal_set)
    }
}

/// Unfolding lemma for `is_connected_via_clopen`.
theorem is_connected_via_clopen_iff[T](open: Set[T] -> Bool) {
    is_connected_via_clopen[T](open) = forall(s: Set[T]) {
        is_clopen[T](open, s) implies (s = Set[T].empty_set or s = Set[T].universal_set)
    }
}

/// The union of all sets in a list of sets. The empty list yields the empty set.
define list_union_of_sets[X](items: List[Set[X]]) -> Set[X] {
    match items {
        List.nil {
            Set[X].empty_set
        }
        List.cons(head, tail) {
            head.union(list_union_of_sets(tail))
        }
    }
}

/// The union of an empty list of sets is the empty set.
theorem list_union_of_sets_nil[X] {
    list_union_of_sets[X](List.nil[Set[X]]) = Set[X].empty_set
}

/// The union of a cons list is the head union the tail union.
theorem list_union_of_sets_cons[X](head: Set[X], tail: List[Set[X]]) {
    list_union_of_sets[X](List.cons(head, tail)) = head.union(list_union_of_sets[X](tail))
}

/// The intersection of all sets in a list of sets. The empty list yields the
/// universal set.
define list_inter_of_sets[X](items: List[Set[X]]) -> Set[X] {
    match items {
        List.nil {
            Set[X].universal_set
        }
        List.cons(head, tail) {
            head.intersection(list_inter_of_sets(tail))
        }
    }
}

/// The intersection of an empty list of sets is the universal set.
theorem list_inter_of_sets_nil[X] {
    list_inter_of_sets[X](List.nil[Set[X]]) = Set[X].universal_set
}

/// The intersection of a cons list is the head intersected with the tail intersection.
theorem list_inter_of_sets_cons[X](head: Set[X], tail: List[Set[X]]) {
    list_inter_of_sets[X](List.cons(head, tail)) = head.intersection(list_inter_of_sets[X](tail))
}

/// True if `c` is an open cover of `s`: every member is open and `s` is contained in the union.
define is_open_cover[T: TopologicalSpace](c: Set[T] -> Bool, s: Set[T]) -> Bool {
    (forall(u: Set[T]) { c(u) implies T.is_open(u) }) and s.subset(big_union(c))
}

/// True if every set in `items` belongs to the family `c`.
define is_subfamily[X](c: Set[X] -> Bool, items: List[Set[X]]) -> Bool {
    match items {
        List.nil {
            true
        }
        List.cons(head, tail) {
            c(head) and is_subfamily(c, tail)
        }
    }
}

/// Empty list is a subfamily of any family.
theorem is_subfamily_nil[X](c: Set[X] -> Bool) {
    is_subfamily[X](c, List.nil[Set[X]])
}

/// Cons unfolding (forward) for `is_subfamily`.
theorem is_subfamily_cons_mp[X](c: Set[X] -> Bool, head: Set[X], tail: List[Set[X]]) {
    is_subfamily[X](c, List.cons(head, tail)) implies (c(head) and is_subfamily[X](c, tail))
}

/// Cons unfolding (backward) for `is_subfamily`.
theorem is_subfamily_cons_mpr[X](c: Set[X] -> Bool, head: Set[X], tail: List[Set[X]]) {
    (c(head) and is_subfamily[X](c, tail)) implies is_subfamily[X](c, List.cons(head, tail))
}


/// Unfolding lemma for `is_open_cover`.
theorem is_open_cover_iff[T: TopologicalSpace](c: Set[T] -> Bool, s: Set[T]) {
    is_open_cover[T](c, s) = ((forall(u: Set[T]) { c(u) implies T.is_open(u) }) and s.subset(big_union(c)))
}

/// True if `s` is compact: every open cover has a finite subcover.
define is_compact[T: TopologicalSpace](s: Set[T]) -> Bool {
    forall(c: Set[T] -> Bool) {
        is_open_cover[T](c, s) implies exists(items: List[Set[T]]) {
            is_subfamily[T](c, items) and s.subset(list_union_of_sets(items))
        }
    }
}

/// Unfolding lemma for `is_compact`.
theorem is_compact_iff[T: TopologicalSpace](s: Set[T]) {
    is_compact[T](s) = forall(c: Set[T] -> Bool) {
        is_open_cover[T](c, s) implies exists(items: List[Set[T]]) {
            is_subfamily[T](c, items) and s.subset(list_union_of_sets[T](items))
        }
    }
}

/// The empty set is compact: the empty list is a finite subcover.
theorem empty_set_is_compact[T: TopologicalSpace] {
    is_compact[T](Set[T].empty_set)
} by {
    forall(c: Set[T] -> Bool) {
        if is_open_cover[T](c, Set[T].empty_set) {
            let items: List[Set[T]] = List.nil[Set[T]]
            is_subfamily[T](c, items) = true
            is_subfamily[T](c, items)
            list_union_of_sets[T](items) = Set[T].empty_set
            empty_set_is_always_subset[T](list_union_of_sets[T](items))
            Set[T].empty_set.subset(list_union_of_sets[T](items))
            is_subfamily[T](c, items) and Set[T].empty_set.subset(list_union_of_sets[T](items))
            exists(its: List[Set[T]]) {
                is_subfamily[T](c, its) and Set[T].empty_set.subset(list_union_of_sets[T](its))
            }
        }
    }
}

/// A point covered by the family `c` has a one-element finite subcover of `{x}`.
theorem singleton_subcover_witness[T: TopologicalSpace](c: Set[T] -> Bool, x: T) {
    big_union(c).contains(x) implies exists(items: List[Set[T]]) {
        is_subfamily[T](c, items) and Set[T].singleton(x).subset(list_union_of_sets[T](items))
    }
} by {
    if big_union(c).contains(x) {
        big_union_contains_eq(c, x)
        let u: Set[T] satisfy {
            c(u) and u.contains(x)
        }
        let items: List[Set[T]] = List.cons(u, List.nil[Set[T]])
        is_subfamily[T](c, items) = (c(u) and is_subfamily[T](c, List.nil[Set[T]]))
        is_subfamily[T](c, List.nil[Set[T]]) = true
        is_subfamily[T](c, items)
        list_union_of_sets[T](items) = u.union(list_union_of_sets[T](List.nil[Set[T]]))
        list_union_of_sets[T](List.nil[Set[T]]) = Set[T].empty_set
        list_union_of_sets[T](items) = u.union(Set[T].empty_set)
        forall(z: T) {
            if Set[T].singleton(x).contains(z) {
                singleton_contains_eq[T](x, z)
                z = x
                u.contains(z)
                union_contains_eq(u, Set[T].empty_set, z)
                u.union(Set[T].empty_set).contains(z)
                list_union_of_sets[T](items).contains(z)
            }
        }
        Set[T].singleton(x).subset(list_union_of_sets[T](items))
        is_subfamily[T](c, items) and Set[T].singleton(x).subset(list_union_of_sets[T](items))
    }
}

/// If a cover covers `{x}`, then `x` lies in the big union of the cover.
theorem singleton_cover_imp_big_union[T: TopologicalSpace](c: Set[T] -> Bool, x: T) {
    is_open_cover[T](c, Set[T].singleton(x)) implies big_union(c).contains(x)
} by {
    if is_open_cover[T](c, Set[T].singleton(x)) {
        is_open_cover[T](c, Set[T].singleton(x)) = (
            (forall(u: Set[T]) { c(u) implies T.is_open(u) })
            and Set[T].singleton(x).subset(big_union(c)))
        Set[T].singleton(x).subset(big_union(c))
        Set[T].singleton(x).subset(big_union(c)) = forall(z: T) {
            Set[T].singleton(x).contains(z) implies big_union(c).contains(z)
        }
        singleton_contains_eq[T](x, x)
        Set[T].singleton(x).contains(x)
        Set[T].singleton(x).contains(x) implies big_union(c).contains(x)
    }
}

/// From an open cover of `{x}`, build a one-element finite subcover.
theorem singleton_open_cover_imp_finite_subcover[T: TopologicalSpace](
    c: Set[T] -> Bool, x: T) {
    is_open_cover[T](c, Set[T].singleton(x)) implies exists(items: List[Set[T]]) {
        is_subfamily[T](c, items) and Set[T].singleton(x).subset(list_union_of_sets[T](items))
    }
} by {
    if is_open_cover[T](c, Set[T].singleton(x)) {
        singleton_cover_imp_big_union[T](c, x)
        singleton_subcover_witness[T](c, x)
    }
}

/// A singleton is compact: every open cover has a one-element finite subcover.
theorem singleton_is_compact[T: TopologicalSpace](x: T) {
    is_compact[T](Set[T].singleton(x))
} by {
    is_compact_iff[T](Set[T].singleton(x))
    if not is_compact[T](Set[T].singleton(x)) {
        let w: (Set[T] -> Bool) satisfy {
            is_open_cover[T](w, Set[T].singleton(x)) and not exists(items: List[Set[T]]) {
                is_subfamily[T](w, items) and Set[T].singleton(x).subset(list_union_of_sets[T](items))
            }
        }
        singleton_open_cover_imp_finite_subcover[T](w, x)
        exists(items: List[Set[T]]) {
            is_subfamily[T](w, items) and Set[T].singleton(x).subset(list_union_of_sets[T](items))
        }
        false
    }
}




/// True if a subset is dense: its closure is the whole space.
define is_dense[T: TopologicalSpace](s: Set[T]) -> Bool {
    closure(s) = Set[T].universal_set
}

/// Universal closure membership makes a set dense.
theorem dense_intro[T: TopologicalSpace](s: Set[T]) {
    (forall(x: T) { closure(s).contains(x) }) implies is_dense[T](s)
} by {
    if forall(x: T) { closure(s).contains(x) } {
        forall(x: T) {
            universal_set_contains_eq[T](x)
            closure(s).contains(x) = Set[T].universal_set.contains(x)
        }
        set_ext(closure(s), Set[T].universal_set)
        closure(s) = Set[T].universal_set
        is_dense[T](s)
    }
}

/// A dense set has every point in its closure.
theorem dense_closure_contains[T: TopologicalSpace](s: Set[T], x: T) {
    is_dense[T](s) implies closure(s).contains(x)
} by {
    if is_dense[T](s) {
        is_dense[T](s) = (closure(s) = Set[T].universal_set)
        closure(s) = Set[T].universal_set
        universal_set_contains_eq[T](x)
        Set[T].universal_set.contains(x)
        closure(s).contains(x)
    }
}

/// The universal set is dense.
theorem universal_set_is_dense[T: TopologicalSpace] {
    is_dense[T](Set[T].universal_set)
} by {
    closure_universal[T]
    closure(Set[T].universal_set) = Set[T].universal_set
    is_dense[T](Set[T].universal_set)
}

/// Enlarging a dense set preserves density.
theorem dense_of_subset[T: TopologicalSpace](s: Set[T], t: Set[T]) {
    s.subset(t) and is_dense[T](s) implies is_dense[T](t)
} by {
    if s.subset(t) and is_dense[T](s) {
        closure_monotone[T](s, t)
        closure(s).subset(closure(t))
        forall(x: T) {
            dense_closure_contains[T](s, x)
            closure(s).contains(x)
            closure(s).subset(closure(t)) = forall(y: T) {
                closure(s).contains(y) implies closure(t).contains(y)
            }
            closure(s).contains(x) implies closure(t).contains(x)
            closure(t).contains(x)
        }
        dense_intro[T](t)
        is_dense[T](t)
    }
}

/// A set is dense exactly when its closure is dense.
theorem dense_closure_eq_dense[T: TopologicalSpace](s: Set[T]) {
    is_dense[T](closure(s)) = is_dense[T](s)
} by {
    closure_idempotent[T](s)
    closure(closure(s)) = closure(s)
    if is_dense[T](closure(s)) {
        is_dense[T](closure(s)) = (closure(closure(s)) = Set[T].universal_set)
        closure(closure(s)) = Set[T].universal_set
        closure(s) = Set[T].universal_set
        is_dense[T](s)
    }
    if is_dense[T](s) {
        is_dense[T](s) = (closure(s) = Set[T].universal_set)
        closure(s) = Set[T].universal_set
        closure(closure(s)) = Set[T].universal_set
        is_dense[T](closure(s))
    }
    is_dense[T](closure(s)) = is_dense[T](s)
}

/// The closure of a dense set is dense.
theorem closure_of_dense_is_dense[T: TopologicalSpace](s: Set[T]) {
    is_dense[T](s) implies is_dense[T](closure(s))
} by {
    if is_dense[T](s) {
        closure_idempotent[T](s)
        closure(closure(s)) = closure(s)
        closure(s) = Set[T].universal_set
        closure(closure(s)) = Set[T].universal_set
        is_dense[T](closure(s))
    }
}

/// Taking a union on the right preserves density of the left member.
theorem dense_union_left[T: TopologicalSpace](s: Set[T], t: Set[T]) {
    is_dense[T](s) implies is_dense[T](s.union(t))
} by {
    if is_dense[T](s) {
        sets_subset_union(s, t)
        s.subset(s.union(t))
        dense_of_subset[T](s, s.union(t))
        is_dense[T](s.union(t))
    }
}

/// Taking a union on the left preserves density of the right member.
theorem dense_union_right[T: TopologicalSpace](s: Set[T], t: Set[T]) {
    is_dense[T](t) implies is_dense[T](s.union(t))
} by {
    if is_dense[T](t) {
        sets_subset_union(s, t)
        t.subset(s.union(t))
        dense_of_subset[T](t, s.union(t))
        is_dense[T](s.union(t))
    }
}

/// True if a subset has empty interior: it contains no open set except the empty set.
define is_nowhere_dense_interior_empty[T: TopologicalSpace](s: Set[T]) -> Bool {
    interior(s) = Set[T].empty_set
}

/// True if a point is an adherent point of `s`: every neighborhood of the point meets `s`.
/// Equivalent to closure membership.
define is_adherent_point[T: TopologicalSpace](x: T, s: Set[T]) -> Bool {
    closure(s).contains(x)
}

/// True if the family `b` covers the whole type: every point lies in some member.
define basis_covers[T](b: Set[T] -> Bool) -> Bool {
    forall(x: T) {
        exists(u: Set[T]) {
            b(u) and u.contains(x)
        }
    }
}

/// True if intersections of two members of `b` are refined by some member around any common point.
define basis_filters[T](b: Set[T] -> Bool) -> Bool {
    forall(u: Set[T], v: Set[T], x: T) {
        b(u) and b(v) and u.contains(x) and v.contains(x) implies exists(w: Set[T]) {
            b(w) and w.contains(x) and w.subset(u.intersection(v))
        }
    }
}

/// True if `b` is a topological basis: it covers the type and is closed under the
/// pointwise refinement of pairwise intersections.
define is_topological_basis[T](b: Set[T] -> Bool) -> Bool {
    basis_covers(b) and basis_filters(b)
}

/// True if `s` is open in the topology generated by `b`: every point of `s` has a
/// member of `b` around it inside `s`.
define generated_open[T](b: Set[T] -> Bool, s: Set[T]) -> Bool {
    forall(x: T) {
        s.contains(x) implies exists(u: Set[T]) {
            b(u) and u.contains(x) and u.subset(s)
        }
    }
}

/// The empty set is open in any generated topology.
theorem generated_open_empty[T](b: Set[T] -> Bool) {
    generated_open(b, Set[T].empty_set)
} by {
    forall(x: T) {
        if (Set[T].empty_set).contains(x) {
            empty_set_contains_eq[T](x)
            false
        }
    }
}

/// The whole space is open in the topology generated by a covering family.
theorem generated_open_universal[T](b: Set[T] -> Bool) {
    basis_covers(b) implies generated_open(b, Set[T].universal_set)
} by {
    if basis_covers(b) {
        forall(x: T) {
            if (Set[T].universal_set).contains(x) {
                exists(u: Set[T]) {
                    b(u) and u.contains(x)
                }
                let u: Set[T] satisfy {
                    b(u) and u.contains(x)
                }
                forall(y: T) {
                    if u.contains(y) {
                        universal_set_contains_eq[T](y)
                        (Set[T].universal_set).contains(y)
                    }
                }
                u.subset(Set[T].universal_set)
                b(u) and u.contains(x) and u.subset(Set[T].universal_set)
            }
        }
    }
}

/// Every member of the basis is open in the generated topology.
theorem basis_member_is_open[T](b: Set[T] -> Bool, u: Set[T]) {
    b(u) implies generated_open(b, u)
} by {
    if b(u) {
        forall(x: T) {
            if u.contains(x) {
                subset_refl(u)
                u.subset(u)
                b(u) and u.contains(x) and u.subset(u)
                exists(w: Set[T]) {
                    b(w) and w.contains(x) and w.subset(u)
                }
            }
        }
    }
}

/// Binary intersections of generated-open sets are generated-open.
theorem generated_open_inter[T](b: Set[T] -> Bool, s: Set[T], t: Set[T]) {
    is_topological_basis(b) and generated_open(b, s) and generated_open(b, t)
        implies generated_open(b, s.intersection(t))
} by {
    if is_topological_basis(b) and generated_open(b, s) and generated_open(b, t) {
        is_topological_basis(b) = (basis_covers(b) and basis_filters(b))
        basis_covers(b) and basis_filters(b)
        basis_filters(b) = forall(u: Set[T], v: Set[T], y: T) {
            b(u) and b(v) and u.contains(y) and v.contains(y) implies exists(w: Set[T]) {
                b(w) and w.contains(y) and w.subset(u.intersection(v))
            }
        }
        generated_open(b, s) = forall(y: T) {
            s.contains(y) implies exists(u: Set[T]) {
                b(u) and u.contains(y) and u.subset(s)
            }
        }
        generated_open(b, t) = forall(y: T) {
            t.contains(y) implies exists(u: Set[T]) {
                b(u) and u.contains(y) and u.subset(t)
            }
        }
        forall(x: T) {
            if s.intersection(t).contains(x) {
                intersection_contains_eq(s, t, x)
                s.contains(x) and t.contains(x)
                s.contains(x) implies exists(u: Set[T]) {
                    b(u) and u.contains(x) and u.subset(s)
                }
                let u: Set[T] satisfy {
                    b(u) and u.contains(x) and u.subset(s)
                }
                t.contains(x) implies exists(v: Set[T]) {
                    b(v) and v.contains(x) and v.subset(t)
                }
                let v: Set[T] satisfy {
                    b(v) and v.contains(x) and v.subset(t)
                }
                b(u) and b(v) and u.contains(x) and v.contains(x)
                b(u) and b(v) and u.contains(x) and v.contains(x) implies exists(w: Set[T]) {
                    b(w) and w.contains(x) and w.subset(u.intersection(v))
                }
                let w: Set[T] satisfy {
                    b(w) and w.contains(x) and w.subset(u.intersection(v))
                }
                forall(y: T) {
                    if w.contains(y) {
                        w.subset(u.intersection(v)) implies (w.contains(y) implies u.intersection(v).contains(y))
                        u.intersection(v).contains(y)
                        intersection_contains_eq(u, v, y)
                        u.contains(y) and v.contains(y)
                        u.subset(s) implies (u.contains(y) implies s.contains(y))
                        s.contains(y)
                        v.subset(t) implies (v.contains(y) implies t.contains(y))
                        t.contains(y)
                        intersection_contains_eq(s, t, y)
                        s.intersection(t).contains(y)
                    }
                }
                w.subset(s.intersection(t))
                b(w) and w.contains(x) and w.subset(s.intersection(t))
                exists(z: Set[T]) {
                    b(z) and z.contains(x) and z.subset(s.intersection(t))
                }
            }
        }
    }
}

/// Arbitrary unions of generated-open sets are generated-open.
theorem generated_open_big_union[T](b: Set[T] -> Bool, c: Set[T] -> Bool) {
    (forall(s: Set[T]) { c(s) implies generated_open(b, s) })
        implies generated_open(b, big_union(c))
} by {
    if forall(s: Set[T]) { c(s) implies generated_open(b, s) } {
        forall(x: T) {
            if big_union(c).contains(x) {
                big_union_contains_eq(c, x)
                let s: Set[T] satisfy {
                    c(s) and s.contains(x)
                }
                generated_open(b, s)
                generated_open(b, s) = forall(y: T) {
                    s.contains(y) implies exists(u: Set[T]) {
                        b(u) and u.contains(y) and u.subset(s)
                    }
                }
                s.contains(x) implies exists(u: Set[T]) {
                    b(u) and u.contains(x) and u.subset(s)
                }
                let u: Set[T] satisfy {
                    b(u) and u.contains(x) and u.subset(s)
                }
                forall(y: T) {
                    if u.contains(y) {
                        u.subset(s) implies (u.contains(y) implies s.contains(y))
                        s.contains(y)
                        c(s) and s.contains(y)
                        big_union_contains_of_member(c, s, y)
                        big_union(c).contains(y)
                    }
                }
                u.subset(big_union(c))
                b(u) and u.contains(x) and u.subset(big_union(c))
                exists(w: Set[T]) {
                    b(w) and w.contains(x) and w.subset(big_union(c))
                }
            }
        }
    }
}

/// The family of basis members contained in a given set.
define basis_subset_family[T](b: Set[T] -> Bool, s: Set[T], u: Set[T]) -> Bool {
    b(u) and u.subset(s)
}

/// A point in a generated-open set has a witnessing basis member contained in the set.
theorem generated_open_witness[T](b: Set[T] -> Bool, s: Set[T], x: T) {
    generated_open(b, s) and s.contains(x) implies exists(u: Set[T]) {
        b(u) and u.contains(x) and u.subset(s)
    }
} by {
    if generated_open(b, s) and s.contains(x) {
        generated_open(b, s) = forall(y: T) {
            s.contains(y) implies exists(u: Set[T]) {
                b(u) and u.contains(y) and u.subset(s)
            }
        }
        forall(y: T) {
            s.contains(y) implies exists(u: Set[T]) {
                b(u) and u.contains(y) and u.subset(s)
            }
        }
        s.contains(x) implies exists(u: Set[T]) {
            b(u) and u.contains(x) and u.subset(s)
        }
    }
}

/// Membership of a basis member of `s` in the basis-subset family.
theorem basis_subset_family_member[T](b: Set[T] -> Bool, s: Set[T], u: Set[T]) {
    b(u) and u.subset(s) implies basis_subset_family(b, s)(u)
} by {
    if b(u) and u.subset(s) {
        basis_subset_family(b, s, u) = (b(u) and u.subset(s))
        basis_subset_family(b, s)(u)
    }
}

/// A member of the basis-subset family is a basis member contained in `s`.
theorem basis_subset_family_member_imp[T](b: Set[T] -> Bool, s: Set[T], u: Set[T]) {
    basis_subset_family(b, s)(u) implies b(u) and u.subset(s)
} by {
    if basis_subset_family(b, s)(u) {
        basis_subset_family(b, s, u) = (b(u) and u.subset(s))
    }
}

/// A direct lifting from a basis-subset witness to big union membership.
theorem big_union_basis_subset_of_witness[T](b: Set[T] -> Bool, s: Set[T], u: Set[T], x: T) {
    b(u) and u.subset(s) and u.contains(x)
        implies big_union(basis_subset_family(b, s)).contains(x)
} by {
    if b(u) and u.subset(s) and u.contains(x) {
        basis_subset_family_member(b, s, u)
        basis_subset_family(b, s)(u)
        big_union_contains_of_member(basis_subset_family(b, s), u, x)
    }
}

/// A point in a generated-open set is in the union of basis members contained in it.
theorem generated_open_subset_imp_big_union_basis_subset[T](b: Set[T] -> Bool, s: Set[T], x: T) {
    generated_open(b, s) and s.contains(x)
        implies big_union(basis_subset_family(b, s)).contains(x)
} by {
    if generated_open(b, s) and s.contains(x) {
        generated_open_witness(b, s, x)
        let u: Set[T] satisfy {
            b(u) and u.contains(x) and u.subset(s)
        }
        b(u)
        u.subset(s)
        u.contains(x)
        b(u) and u.subset(s) and u.contains(x)
        big_union_basis_subset_of_witness(b, s, u, x)
        big_union(basis_subset_family(b, s)).contains(x)
    }
}

/// A point in the union of basis members of a set is in the set.
theorem big_union_basis_subset_subset[T](b: Set[T] -> Bool, s: Set[T], x: T) {
    big_union(basis_subset_family(b, s)).contains(x) implies s.contains(x)
} by {
    if big_union(basis_subset_family(b, s)).contains(x) {
        let f = basis_subset_family(b, s)
        big_union_contains_eq(f, x)
        let u: Set[T] satisfy {
            f(u) and u.contains(x)
        }
        basis_subset_family_member_imp(b, s, u)
        b(u) and u.subset(s)
        u.subset(s) implies (u.contains(x) implies s.contains(x))
        s.contains(x)
    }
}

/// Pointwise membership equality for the basis-subset characterization.
theorem generated_open_basis_subset_pointwise[T](b: Set[T] -> Bool, s: Set[T], x: T) {
    generated_open(b, s) implies s.contains(x) = big_union(basis_subset_family(b, s)).contains(x)
} by {
    if generated_open(b, s) {
        if s.contains(x) {
            generated_open_subset_imp_big_union_basis_subset(b, s, x)
            big_union(basis_subset_family(b, s)).contains(x)
        }
        if big_union(basis_subset_family(b, s)).contains(x) {
            big_union_basis_subset_subset(b, s, x)
            s.contains(x)
        }
    }
}

/// A generated-open set equals the union of the basis members contained in it.
theorem generated_open_eq_big_union_basis_subset[T](b: Set[T] -> Bool, s: Set[T]) {
    generated_open(b, s) implies s = big_union(basis_subset_family(b, s))
} by {
    if generated_open(b, s) {
        forall(x: T) {
            generated_open_basis_subset_pointwise(b, s, x)
            s.contains(x) = big_union(basis_subset_family(b, s)).contains(x)
        }
        set_ext(s, big_union(basis_subset_family(b, s)))
    }
}

/// The union of any sub-family of basis members is generated-open.
theorem big_union_basis_subfamily_generated_open[T](b: Set[T] -> Bool, c: Set[T] -> Bool) {
    (forall(u: Set[T]) { c(u) implies b(u) })
        implies generated_open(b, big_union(c))
} by {
    if forall(u: Set[T]) { c(u) implies b(u) } {
        forall(u: Set[T]) {
            if c(u) {
                c(u) implies b(u)
                b(u)
                basis_member_is_open(b, u)
                generated_open(b, u)
            }
        }
        generated_open_big_union[T](b, c)
        generated_open(b, big_union(c))
    }
}

/// Generated-open characterisation: a set is generated-open iff it is the union of
/// some sub-family of the basis.
theorem generated_open_iff_union_of_basis[T](b: Set[T] -> Bool, s: Set[T]) {
    generated_open(b, s) = exists(c: Set[T] -> Bool) {
        (forall(u: Set[T]) { c(u) implies b(u) }) and s = big_union(c)
    }
} by {
    if generated_open(b, s) {
        generated_open_eq_big_union_basis_subset(b, s)
        s = big_union(basis_subset_family(b, s))
        let f = basis_subset_family(b, s)
        forall(u: Set[T]) {
            if f(u) {
                basis_subset_family(b, s, u) = (b(u) and u.subset(s))
                f(u) = (b(u) and u.subset(s))
                b(u)
            }
        }
        (forall(u: Set[T]) { f(u) implies b(u) }) and s = big_union(f)
        exists(c: Set[T] -> Bool) {
            (forall(u: Set[T]) { c(u) implies b(u) }) and s = big_union(c)
        }
    }
    if exists(c: Set[T] -> Bool) {
        (forall(u: Set[T]) { c(u) implies b(u) }) and s = big_union(c)
    } {
        let c: Set[T] -> Bool satisfy {
            (forall(u: Set[T]) { c(u) implies b(u) }) and s = big_union(c)
        }
        big_union_basis_subfamily_generated_open(b, c)
        generated_open(b, big_union(c))
        s = big_union(c)
        generated_open(b, s)
    }
}

/// A simple type used as an example of a discrete topological space.
inductive TopoExample {
    /// First sample point.
    a

    /// Second sample point.
    b
}

/// The discrete topology on TopoExample: every set is open.
define topo_example_open(s: Set[TopoExample]) -> Bool {
    true
}

instance TopoExample: TopologicalSpace {
    let is_open: Set[TopoExample] -> Bool = topo_example_open
}

/// True if `n` is a neighborhood of `x`: an open set containing `x` is contained in `n`.
define is_neighborhood[T: TopologicalSpace](n: Set[T], x: T) -> Bool {
    exists(u: Set[T]) {
        T.is_open(u) and u.contains(x) and u.subset(n)
    }
}

/// Every neighborhood of `x` contains `x`.
theorem neighborhood_contains_point[T: TopologicalSpace](n: Set[T], x: T) {
    is_neighborhood(n, x) implies n.contains(x)
} by {
    if is_neighborhood(n, x) {
        let u: Set[T] satisfy {
            T.is_open(u) and u.contains(x) and u.subset(n)
        }
        u.subset(n) implies (u.contains(x) implies n.contains(x))
        n.contains(x)
    }
}

/// The whole space is a neighborhood of every point.
theorem universal_is_neighborhood[T: TopologicalSpace](x: T) {
    is_neighborhood(Set[T].universal_set, x)
} by {
    let u = Set[T].universal_set
    open_universal[T]
    T.is_open(u)
    universal_set_contains_eq[T](x)
    u.contains(x)
    subset_refl(u)
    u.subset(u)
    T.is_open(u) and u.contains(x) and u.subset(u)
}

/// An open set is a neighborhood of every point it contains.
theorem open_is_neighborhood[T: TopologicalSpace](u: Set[T], x: T) {
    T.is_open(u) and u.contains(x) implies is_neighborhood(u, x)
} by {
    if T.is_open(u) and u.contains(x) {
        subset_refl(u)
        u.subset(u)
        T.is_open(u) and u.contains(x) and u.subset(u)
        exists(v: Set[T]) {
            T.is_open(v) and v.contains(x) and v.subset(u)
        }
    }
}

/// A superset of a neighborhood is a neighborhood.
theorem neighborhood_superset[T: TopologicalSpace](n: Set[T], m: Set[T], x: T) {
    is_neighborhood(n, x) and n.subset(m) implies is_neighborhood(m, x)
} by {
    if is_neighborhood(n, x) and n.subset(m) {
        let u: Set[T] satisfy {
            T.is_open(u) and u.contains(x) and u.subset(n)
        }
        forall(y: T) {
            if u.contains(y) {
                u.subset(n) implies (u.contains(y) implies n.contains(y))
                n.contains(y)
                n.subset(m) implies (n.contains(y) implies m.contains(y))
                m.contains(y)
            }
        }
        u.subset(m)
        T.is_open(u) and u.contains(x) and u.subset(m)
        exists(v: Set[T]) {
            T.is_open(v) and v.contains(x) and v.subset(m)
        }
    }
}

/// The intersection of two neighborhoods is a neighborhood.
theorem neighborhood_inter[T: TopologicalSpace](n: Set[T], m: Set[T], x: T) {
    is_neighborhood(n, x) and is_neighborhood(m, x)
        implies is_neighborhood(n.intersection(m), x)
} by {
    if is_neighborhood(n, x) and is_neighborhood(m, x) {
        let u: Set[T] satisfy {
            T.is_open(u) and u.contains(x) and u.subset(n)
        }
        let v: Set[T] satisfy {
            T.is_open(v) and v.contains(x) and v.subset(m)
        }
        open_inter[T](u, v)
        T.is_open(u.intersection(v))
        intersection_contains_eq(u, v, x)
        u.intersection(v).contains(x)
        forall(y: T) {
            if u.intersection(v).contains(y) {
                intersection_contains_eq(u, v, y)
                u.contains(y) and v.contains(y)
                u.subset(n) implies (u.contains(y) implies n.contains(y))
                n.contains(y)
                v.subset(m) implies (v.contains(y) implies m.contains(y))
                m.contains(y)
                intersection_contains_eq(n, m, y)
                n.intersection(m).contains(y)
            }
        }
        u.intersection(v).subset(n.intersection(m))
        T.is_open(u.intersection(v)) and u.intersection(v).contains(x) and u.intersection(v).subset(n.intersection(m))
        exists(w: Set[T]) {
            T.is_open(w) and w.contains(x) and w.subset(n.intersection(m))
        }
    }
}

/// A point is in the interior of `s` exactly when `s` is a neighborhood of the point.
theorem interior_iff_neighborhood[T: TopologicalSpace](s: Set[T], x: T) {
    interior(s).contains(x) = is_neighborhood(s, x)
} by {
    if interior(s).contains(x) {
        interior_contains_eq(s, x)
        let u: Set[T] satisfy {
            T.is_open(u) and u.subset(s) and u.contains(x)
        }
        T.is_open(u) and u.contains(x) and u.subset(s)
        exists(v: Set[T]) {
            T.is_open(v) and v.contains(x) and v.subset(s)
        }
        is_neighborhood(s, x)
    }
    if is_neighborhood(s, x) {
        let u: Set[T] satisfy {
            T.is_open(u) and u.contains(x) and u.subset(s)
        }
        T.is_open(u) and u.subset(s) and u.contains(x)
        exists(v: Set[T]) {
            T.is_open(v) and v.subset(s) and v.contains(x)
        }
        interior_contains_eq(s, x)
        interior(s).contains(x)
    }
}

/// A set is open iff it is a neighborhood of each of its points.
theorem open_iff_neighborhood_of_each_point[T: TopologicalSpace](s: Set[T]) {
    T.is_open(s) = forall(x: T) {
        s.contains(x) implies is_neighborhood(s, x)
    }
} by {
    if T.is_open(s) {
        forall(x: T) {
            if s.contains(x) {
                open_is_neighborhood[T](s, x)
                is_neighborhood(s, x)
            }
        }
    }
    if forall(x: T) { s.contains(x) implies is_neighborhood(s, x) } {
        forall(x: T) {
            if s.contains(x) {
                is_neighborhood(s, x)
                interior_iff_neighborhood[T](s, x)
                interior(s).contains(x)
            }
        }
        s.subset(interior(s))
        interior_subset[T](s)
        interior(s).subset(s)
        subset_antisymm(s, interior(s))
        s = interior(s)
        interior_open[T](s)
        T.is_open(interior(s))
        T.is_open(s)
    }
}

/// A point lies in the closure of `s` iff every neighborhood of the point meets `s`.
theorem closure_iff_every_neighborhood_meets[T: TopologicalSpace](s: Set[T], x: T) {
    closure(s).contains(x) = forall(n: Set[T]) {
        is_neighborhood(n, x) implies exists(y: T) {
            n.contains(y) and s.contains(y)
        }
    }
} by {
    if closure(s).contains(x) {
        forall(n: Set[T]) {
            if is_neighborhood(n, x) {
                let u: Set[T] satisfy {
                    T.is_open(u) and u.contains(x) and u.subset(n)
                }
                let c = u.c
                T.is_open(u)
                is_closed(c) = T.is_open(c.c)
                compl_of_compl_is_self(u)
                u.c.c = u
                T.is_open(u.c.c)
                T.is_open(c.c)
                is_closed(c)
                if s.subset(c) {
                    closure_contains_imp_member[T](s, c, x)
                    c.contains(x)
                    compl_contains_eq(u, x)
                    u.c.contains(x)
                    not u.contains(x)
                    false
                }
                not s.subset(c)
                exists(y: T) {
                    s.contains(y) and not c.contains(y)
                }
                let y: T satisfy {
                    s.contains(y) and not c.contains(y)
                }
                compl_contains_eq(u, y)
                u.c.contains(y) = not u.contains(y)
                not c.contains(y)
                c = u.c
                not u.c.contains(y)
                u.contains(y)
                u.subset(n) implies (u.contains(y) implies n.contains(y))
                n.contains(y)
                n.contains(y) and s.contains(y)
                exists(z: T) {
                    n.contains(z) and s.contains(z)
                }
            }
        }
    }
    if forall(n: Set[T]) { is_neighborhood(n, x) implies exists(y: T) { n.contains(y) and s.contains(y) } } {
        forall(c: Set[T]) {
            if is_closed(c) and s.subset(c) {
                T.is_open(c.c)
                if c.c.contains(x) {
                    open_is_neighborhood[T](c.c, x)
                    is_neighborhood(c.c, x)
                    is_neighborhood(c.c, x) implies exists(y: T) {
                        c.c.contains(y) and s.contains(y)
                    }
                    let y: T satisfy {
                        c.c.contains(y) and s.contains(y)
                    }
                    s.subset(c) implies (s.contains(y) implies c.contains(y))
                    c.contains(y)
                    compl_contains_eq(c, y)
                    not c.contains(y)
                    false
                }
                not c.c.contains(x)
                compl_contains_eq(c, x)
                c.contains(x)
            }
        }
        closure_contains_eq(s, x)
        closure(s).contains(x)
    }
}

/// True if a point is a limit point of `s`: every neighborhood of the point contains a
/// point of `s` distinct from the point itself.
define is_limit_point[T: TopologicalSpace](x: T, s: Set[T]) -> Bool {
    forall(n: Set[T]) {
        is_neighborhood(n, x) implies exists(y: T) {
            n.contains(y) and s.contains(y) and y != x
        }
    }
}

/// True if a point of `s` is isolated in `s`: it has a neighborhood that meets `s` only at itself.
define is_isolated_point[T: TopologicalSpace](x: T, s: Set[T]) -> Bool {
    s.contains(x) and exists(n: Set[T]) {
        is_neighborhood(n, x) and forall(y: T) {
            n.contains(y) and s.contains(y) implies y = x
        }
    }
}

/// True if `f` is continuous at the point `x`: every neighborhood of `f(x)` has a
/// neighborhood-of-`x` preimage.
define is_continuous_at[X: TopologicalSpace, Y: TopologicalSpace](f: X -> Y, x: X) -> Bool {
    forall(v: Set[Y]) {
        is_neighborhood(v, f(x)) implies is_neighborhood(set_preimage(f, v), x)
    }
}

/// If `f` is continuous at every point and the preimage of an open `v` contains `x`,
/// then the preimage is a neighborhood of `x`.
theorem pointwise_continuous_preimage_neighborhood[X: TopologicalSpace, Y: TopologicalSpace](
    f: X -> Y, v: Set[Y], x: X) {
    (forall(y: X) { is_continuous_at[X, Y](f, y) })
        and Y.is_open(v) and set_preimage(f, v).contains(x)
        implies is_neighborhood(set_preimage(f, v), x)
} by {
    if (forall(y: X) { is_continuous_at[X, Y](f, y) })
        and Y.is_open(v) and set_preimage(f, v).contains(x) {
        set_preimage_contains_eq(f, v, x)
        preimage_contains(f, v, x) = v.contains(f(x))
        v.contains(f(x))
        open_is_neighborhood[Y](v, f(x))
        is_neighborhood(v, f(x))
        is_continuous_at[X, Y](f, x)
        is_continuous_at[X, Y](f, x) = forall(w: Set[Y]) {
            is_neighborhood(w, f(x)) implies is_neighborhood(set_preimage(f, w), x)
        }
        is_neighborhood(v, f(x)) implies is_neighborhood(set_preimage(f, v), x)
    }
}

/// A globally continuous function is continuous at every point.
theorem continuous_imp_continuous_at[X: TopologicalSpace, Y: TopologicalSpace](
    f: X -> Y, x: X) {
    is_continuous(f) implies is_continuous_at[X, Y](f, x)
} by {
    if is_continuous(f) {
        is_continuous(f) = forall(w: Set[Y]) {
            Y.is_open(w) implies X.is_open(set_preimage(f, w))
        }
        forall(v: Set[Y]) {
            if is_neighborhood(v, f(x)) {
                let w: Set[Y] satisfy {
                    Y.is_open(w) and w.contains(f(x)) and w.subset(v)
                }
                Y.is_open(w) implies X.is_open(set_preimage(f, w))
                X.is_open(set_preimage(f, w))
                set_preimage_contains_eq(f, w, x)
                preimage_contains(f, w, x) = w.contains(f(x))
                set_preimage(f, w).contains(x) = w.contains(f(x))
                set_preimage(f, w).contains(x)
                forall(y: X) {
                    if set_preimage(f, w).contains(y) {
                        set_preimage_contains_eq(f, w, y)
                        preimage_contains(f, w, y) = w.contains(f(y))
                        w.contains(f(y))
                        w.subset(v) implies (w.contains(f(y)) implies v.contains(f(y)))
                        v.contains(f(y))
                        set_preimage_contains_eq(f, v, y)
                        preimage_contains(f, v, y) = v.contains(f(y))
                        set_preimage(f, v).contains(y)
                    }
                }
                set_preimage(f, w).subset(set_preimage(f, v))
                X.is_open(set_preimage(f, w)) and set_preimage(f, w).contains(x) and set_preimage(f, w).subset(set_preimage(f, v))
                exists(u: Set[X]) {
                    X.is_open(u) and u.contains(x) and u.subset(set_preimage(f, v))
                }
            }
        }
    }
}

/// If `f` is continuous at every point, then `f` is globally continuous.
theorem continuous_at_imp_continuous[X: TopologicalSpace, Y: TopologicalSpace](
    f: X -> Y) {
    (forall(x: X) { is_continuous_at[X, Y](f, x) }) implies is_continuous(f)
} by {
    if forall(x: X) { is_continuous_at[X, Y](f, x) } {
        is_continuous(f) = forall(v: Set[Y]) {
            Y.is_open(v) implies X.is_open(set_preimage(f, v))
        }
        forall(v: Set[Y]) {
            if Y.is_open(v) {
                forall(x: X) {
                    if set_preimage(f, v).contains(x) {
                        pointwise_continuous_preimage_neighborhood[X, Y](f, v, x)
                        is_neighborhood(set_preimage(f, v), x)
                    }
                }
                open_iff_neighborhood_of_each_point[X](set_preimage(f, v))
                X.is_open(set_preimage(f, v))
            }
        }
    }
}

/// A function is continuous iff it is continuous at every point.
theorem continuous_iff_continuous_at[X: TopologicalSpace, Y: TopologicalSpace](
    f: X -> Y) {
    is_continuous(f) = forall(x: X) { is_continuous_at[X, Y](f, x) }
} by {
    if is_continuous(f) {
        forall(x: X) {
            continuous_imp_continuous_at[X, Y](f, x)
        }
    }
    if forall(x: X) { is_continuous_at[X, Y](f, x) } {
        continuous_at_imp_continuous[X, Y](f)
    }
}

/// True if the point `x` has a compact neighborhood. A space where every point has a
/// compact neighborhood is locally compact.
define has_compact_neighborhood[T: TopologicalSpace](x: T) -> Bool {
    exists(n: Set[T]) {
        is_neighborhood(n, x) and is_compact[T](n)
    }
}

/// If the universal set is compact, then every point has a compact neighborhood.
theorem universal_compact_imp_has_compact_neighborhood[T: TopologicalSpace](x: T) {
    is_compact[T](Set[T].universal_set) implies has_compact_neighborhood[T](x)
} by {
    if is_compact[T](Set[T].universal_set) {
        universal_is_neighborhood[T](x)
        is_neighborhood(Set[T].universal_set, x)
        is_neighborhood(Set[T].universal_set, x) and is_compact[T](Set[T].universal_set)
        exists(n: Set[T]) {
            is_neighborhood(n, x) and is_compact[T](n)
        }
    }
}

/// The neighborhood filter at `x`: the family of all neighborhoods of `x`.
define neighborhood_filter[T: TopologicalSpace](x: T, n: Set[T]) -> Bool {
    is_neighborhood(n, x)
}

/// Membership in the neighborhood filter is exactly being a neighborhood.
theorem neighborhood_filter_iff[T: TopologicalSpace](x: T, n: Set[T]) {
    neighborhood_filter(x, n) = is_neighborhood(n, x)
}

/// The whole space is in the neighborhood filter at every point.
theorem neighborhood_filter_universal[T: TopologicalSpace](x: T) {
    neighborhood_filter(x, Set[T].universal_set)
} by {
    universal_is_neighborhood[T](x)
    is_neighborhood(Set[T].universal_set, x)
    neighborhood_filter_iff(x, Set[T].universal_set)
}

/// The neighborhood filter is upward closed: any superset of a neighborhood is a neighborhood.
theorem neighborhood_filter_upward_closed[T: TopologicalSpace](x: T, n: Set[T], m: Set[T]) {
    neighborhood_filter(x, n) and n.subset(m) implies neighborhood_filter(x, m)
} by {
    if neighborhood_filter(x, n) and n.subset(m) {
        neighborhood_filter_iff(x, n)
        is_neighborhood(n, x)
        neighborhood_superset[T](n, m, x)
        is_neighborhood(m, x)
        neighborhood_filter_iff(x, m)
    }
}

/// The neighborhood filter is closed under binary intersection.
theorem neighborhood_filter_inter[T: TopologicalSpace](x: T, n: Set[T], m: Set[T]) {
    is_neighborhood(n, x) and is_neighborhood(m, x)
        implies neighborhood_filter(x, n.intersection(m))
} by {
    if is_neighborhood(n, x) and is_neighborhood(m, x) {
        neighborhood_inter[T](n, m, x)
        is_neighborhood(n.intersection(m), x)
        neighborhood_filter_iff(x, n.intersection(m))
    }
}

/// Every member of the neighborhood filter contains `x`.
theorem neighborhood_filter_contains_point[T: TopologicalSpace](x: T, n: Set[T]) {
    neighborhood_filter(x, n) implies n.contains(x)
} by {
    if neighborhood_filter(x, n) {
        neighborhood_filter_iff(x, n)
        is_neighborhood(n, x)
        neighborhood_contains_point[T](n, x)
    }
}

/// The empty set is never in the neighborhood filter (the filter is proper).
theorem neighborhood_filter_excludes_empty[T: TopologicalSpace](x: T) {
    not neighborhood_filter(x, Set[T].empty_set)
} by {
    if neighborhood_filter(x, Set[T].empty_set) {
        neighborhood_filter_contains_point[T](x, Set[T].empty_set)
        Set[T].empty_set.contains(x)
        empty_set_contains_eq[T](x)
        not Set[T].empty_set.contains(x)
        false
    }
}

/// Every open set containing `x` is in the neighborhood filter at `x`.
theorem neighborhood_filter_open_member[T: TopologicalSpace](x: T, u: Set[T]) {
    T.is_open(u) and u.contains(x) implies neighborhood_filter(x, u)
} by {
    if T.is_open(u) and u.contains(x) {
        open_is_neighborhood[T](u, x)
        is_neighborhood(u, x)
        neighborhood_filter_iff(x, u)
    }
}

/// A set is open iff it lies in the neighborhood filter of each of its points.
theorem open_iff_in_neighborhood_filter_of_each_point[T: TopologicalSpace](s: Set[T]) {
    T.is_open(s) = forall(x: T) {
        s.contains(x) implies neighborhood_filter(x, s)
    }
} by {
    open_iff_neighborhood_of_each_point[T](s)
    if T.is_open(s) {
        forall(x: T) {
            if s.contains(x) {
                neighborhood_filter_open_member[T](x, s)
                neighborhood_filter(x, s)
            }
        }
    }
    if forall(x: T) { s.contains(x) implies neighborhood_filter(x, s) } {
        forall(x: T) {
            if s.contains(x) {
                neighborhood_filter(x, s)
                neighborhood_filter_iff(x, s)
                is_neighborhood(s, x)
            }
        }
        T.is_open(s)
    }
}

/// A point lies in the interior of `s` iff `s` is in the neighborhood filter of the point.
theorem interior_iff_in_neighborhood_filter[T: TopologicalSpace](s: Set[T], x: T) {
    interior(s).contains(x) = neighborhood_filter(x, s)
} by {
    interior_iff_neighborhood[T](s, x)
    interior(s).contains(x) = is_neighborhood(s, x)
    neighborhood_filter_iff(x, s)
    is_neighborhood(s, x) = neighborhood_filter(x, s)
}

/// A point lies in the closure of `s` iff every member of the neighborhood filter meets `s`.
theorem closure_iff_every_filter_member_meets[T: TopologicalSpace](s: Set[T], x: T) {
    closure(s).contains(x) = forall(n: Set[T]) {
        neighborhood_filter(x, n) implies exists(y: T) {
            n.contains(y) and s.contains(y)
        }
    }
} by {
    closure_iff_every_neighborhood_meets[T](s, x)
    if closure(s).contains(x) {
        forall(n: Set[T]) {
            if neighborhood_filter(x, n) {
                neighborhood_filter_iff(x, n)
                is_neighborhood(n, x)
                exists(y: T) {
                    n.contains(y) and s.contains(y)
                }
            }
        }
    }
    if forall(n: Set[T]) {
        neighborhood_filter(x, n) implies exists(y: T) {
            n.contains(y) and s.contains(y)
        }
    } {
        forall(c: Set[T]) {
            if is_closed(c) and s.subset(c) {
                T.is_open(c.c)
                if c.c.contains(x) {
                    open_is_neighborhood[T](c.c, x)
                    is_neighborhood(c.c, x)
                    neighborhood_filter_iff(x, c.c)
                    neighborhood_filter(x, c.c)
                    neighborhood_filter(x, c.c) implies exists(y: T) {
                        c.c.contains(y) and s.contains(y)
                    }
                    let y: T satisfy {
                        c.c.contains(y) and s.contains(y)
                    }
                    subset_contains(s, c, y)
                    c.contains(y)
                    compl_contains_eq(c, y)
                    not c.contains(y)
                    false
                }
                not c.c.contains(x)
                compl_contains_eq(c, x)
                c.contains(x)
            }
        }
        closure_contains_eq(s, x)
        closure(s).contains(x)
    }
}

/// A point lies in the closure of `s` iff it is an adherent point of `s`.
theorem closure_iff_adherent[T: TopologicalSpace](s: Set[T], x: T) {
    closure(s).contains(x) = is_adherent_point[T](x, s)
} by {
    is_adherent_point[T](x, s) = closure(s).contains(x)
}

/// A point in the set is adherent.
theorem in_set_imp_adherent[T: TopologicalSpace](s: Set[T], x: T) {
    s.contains(x) implies is_adherent_point[T](x, s)
} by {
    if s.contains(x) {
        subset_closure(s)
        s.subset(closure(s))
        s.subset(closure(s)) implies (s.contains(x) implies closure(s).contains(x))
        closure(s).contains(x)
        is_adherent_point[T](x, s) = closure(s).contains(x)
    }
}

/// Helper: every-neighborhood-meets-set follows from limit-point.
theorem limit_point_imp_every_neighborhood_meets[T: TopologicalSpace](s: Set[T], x: T) {
    is_limit_point[T](x, s) implies forall(n: Set[T]) {
        is_neighborhood(n, x) implies exists(y: T) {
            n.contains(y) and s.contains(y)
        }
    }
} by {
    if is_limit_point[T](x, s) {
        is_limit_point[T](x, s) = forall(n: Set[T]) {
            is_neighborhood(n, x) implies exists(y: T) {
                n.contains(y) and s.contains(y) and y != x
            }
        }
        forall(n: Set[T]) {
            if is_neighborhood(n, x) {
                is_neighborhood(n, x) implies exists(y: T) {
                    n.contains(y) and s.contains(y) and y != x
                }
                let y: T satisfy {
                    n.contains(y) and s.contains(y) and y != x
                }
                exists(z: T) {
                    n.contains(z) and s.contains(z)
                }
            }
        }
    }
}

/// A limit point is an adherent point.
theorem limit_point_imp_adherent[T: TopologicalSpace](s: Set[T], x: T) {
    is_limit_point[T](x, s) implies is_adherent_point[T](x, s)
} by {
    if is_limit_point[T](x, s) {
        limit_point_imp_every_neighborhood_meets[T](s, x)
        closure_iff_every_neighborhood_meets[T](s, x)
        closure(s).contains(x)
        is_adherent_point[T](x, s) = closure(s).contains(x)
    }
}

/// An adherent point that is not in the set is a limit point.
theorem adherent_not_in_set_imp_limit_point[T: TopologicalSpace](s: Set[T], x: T) {
    is_adherent_point[T](x, s) and not s.contains(x) implies is_limit_point[T](x, s)
} by {
    if is_adherent_point[T](x, s) and not s.contains(x) {
        is_adherent_point[T](x, s) = closure(s).contains(x)
        closure(s).contains(x)
        closure_iff_every_neighborhood_meets[T](s, x)
        forall(n: Set[T]) {
            if is_neighborhood(n, x) {
                exists(y: T) {
                    n.contains(y) and s.contains(y)
                }
                let y: T satisfy {
                    n.contains(y) and s.contains(y)
                }
                if y = x {
                    s.contains(y)
                    s.contains(x)
                    false
                }
                y != x
                n.contains(y) and s.contains(y) and y != x
                exists(z: T) {
                    n.contains(z) and s.contains(z) and z != x
                }
            }
        }
    }
}

/// Closure-via-limit-points: a point is in the closure of `s` iff it is in `s` or a limit point of `s`.
theorem closure_iff_in_set_or_limit_point[T: TopologicalSpace](s: Set[T], x: T) {
    closure(s).contains(x) = (s.contains(x) or is_limit_point[T](x, s))
} by {
    if closure(s).contains(x) {
        is_adherent_point[T](x, s) = closure(s).contains(x)
        is_adherent_point[T](x, s)
        if s.contains(x) {
            s.contains(x) or is_limit_point[T](x, s)
        }
        if not s.contains(x) {
            adherent_not_in_set_imp_limit_point[T](s, x)
            is_limit_point[T](x, s)
            s.contains(x) or is_limit_point[T](x, s)
        }
    }
    if s.contains(x) {
        in_set_imp_adherent[T](s, x)
        is_adherent_point[T](x, s) = closure(s).contains(x)
        closure(s).contains(x)
    }
    if is_limit_point[T](x, s) {
        limit_point_imp_adherent[T](s, x)
        is_adherent_point[T](x, s)
        is_adherent_point[T](x, s) = closure(s).contains(x)
        closure(s).contains(x)
    }
}


/// A subset is dense iff its complement has empty interior.
theorem dense_iff_compl_interior_empty[T: TopologicalSpace](s: Set[T]) {
    is_dense[T](s) = (interior(s.c) = Set[T].empty_set)
} by {
    compl_closure_eq_interior_compl[T](s)
    closure(s).c = interior(s.c)
    if is_dense[T](s) {
        is_dense[T](s) = (closure(s) = Set[T].universal_set)
        closure(s) = Set[T].universal_set
        universal_set_compl_is_empty[T]
        Set[T].universal_set.c = Set[T].empty_set
        closure(s).c = Set[T].universal_set.c
        closure(s).c = Set[T].empty_set
        interior(s.c) = Set[T].empty_set
    }
    if interior(s.c) = Set[T].empty_set {
        closure(s).c = Set[T].empty_set
        compl_of_compl_is_self(closure(s))
        closure(s).c.c = closure(s)
        empty_set_compl_is_universal[T]
        Set[T].empty_set.c = Set[T].universal_set
        closure(s).c.c = Set[T].empty_set.c
        closure(s) = Set[T].universal_set
        is_dense[T](s) = (closure(s) = Set[T].universal_set)
    }
}

/// A subset is dense iff every open set containing a point meets it.
theorem dense_iff_every_nonempty_open_meets[T: TopologicalSpace](s: Set[T]) {
    is_dense[T](s) = forall(u: Set[T], x: T) {
        T.is_open(u) and u.contains(x) implies exists(y: T) {
            u.contains(y) and s.contains(y)
        }
    }
} by {
    if is_dense[T](s) {
        closure(s) = Set[T].universal_set
        forall(u: Set[T], x: T) {
            if T.is_open(u) and u.contains(x) {
                open_is_neighborhood[T](u, x)
                is_neighborhood(u, x)
                universal_set_contains_eq[T](x)
                closure(s).contains(x)
                closure_iff_every_neighborhood_meets[T](s, x)
                exists(y: T) {
                    u.contains(y) and s.contains(y)
                }
            }
        }
    }
    if forall(u: Set[T], x: T) {
        T.is_open(u) and u.contains(x) implies exists(y: T) {
            u.contains(y) and s.contains(y)
        }
    } {
        forall(x: T) {
            forall(c: Set[T]) {
                if is_closed(c) and s.subset(c) {
                    T.is_open(c.c)
                    if c.c.contains(x) {
                        T.is_open(c.c) and c.c.contains(x)
                        let z: T satisfy {
                            c.c.contains(z) and s.contains(z)
                        }
                        s.subset(c) implies (s.contains(z) implies c.contains(z))
                        c.contains(z)
                        compl_contains_eq(c, z)
                        not c.contains(z)
                        false
                    }
                    not c.c.contains(x)
                    compl_contains_eq(c, x)
                    c.contains(x)
                }
            }
            closure_contains_eq(s, x)
            closure(s).contains(x)
            universal_set_contains_eq[T](x)
            Set[T].universal_set.contains(x)
            closure(s).contains(x) = Set[T].universal_set.contains(x)
        }
        set_ext(closure(s), Set[T].universal_set)
        closure(s) = Set[T].universal_set
    }
}

/// True if `u` is open in the subspace `a` of the ambient topological space `T`:
/// `u` is the intersection of some ambient open set with `a`.
define is_open_in_subspace[T: TopologicalSpace](a: Set[T], u: Set[T]) -> Bool {
    exists(v: Set[T]) {
        T.is_open(v) and u = v.intersection(a)
    }
}

/// The empty set is open in any subspace.
theorem subspace_open_empty[T: TopologicalSpace](a: Set[T]) {
    is_open_in_subspace[T](a, Set[T].empty_set)
} by {
    let e = Set[T].empty_set
    open_empty[T]
    T.is_open(e)
    forall(x: T) {
        intersection_contains_eq(e, a, x)
        empty_set_contains_eq[T](x)
        e.intersection(a).contains(x) = e.contains(x)
    }
    set_ext(e.intersection(a), e)
    e.intersection(a) = e
    e = e.intersection(a)
    T.is_open(e) and e = e.intersection(a)
    exists(v: Set[T]) {
        T.is_open(v) and e = v.intersection(a)
    }
}

/// The whole subspace `a` is open in itself.
theorem subspace_open_self[T: TopologicalSpace](a: Set[T]) {
    is_open_in_subspace[T](a, a)
} by {
    open_universal[T]
    T.is_open(Set[T].universal_set)
    forall(x: T) {
        intersection_contains_eq(Set[T].universal_set, a, x)
        universal_set_contains_eq[T](x)
        Set[T].universal_set.intersection(a).contains(x) = a.contains(x)
    }
    set_ext(Set[T].universal_set.intersection(a), a)
    Set[T].universal_set.intersection(a) = a
    a = Set[T].universal_set.intersection(a)
    T.is_open(Set[T].universal_set) and a = Set[T].universal_set.intersection(a)
    exists(v: Set[T]) {
        T.is_open(v) and a = v.intersection(a)
    }
}

/// Helper: the intersection of two trace sets distributes through ambient intersection.
theorem subspace_inter_distrib[T](v: Set[T], w: Set[T], a: Set[T]) {
    v.intersection(a).intersection(w.intersection(a)) = v.intersection(w).intersection(a)
} by {
    let l = v.intersection(a).intersection(w.intersection(a))
    let r = v.intersection(w).intersection(a)
    forall(x: T) {
        intersection_contains_eq(v.intersection(a), w.intersection(a), x)
        intersection_contains_eq(v, a, x)
        intersection_contains_eq(w, a, x)
        intersection_contains_eq(v.intersection(w), a, x)
        intersection_contains_eq(v, w, x)
        if l.contains(x) {
            v.intersection(a).contains(x) and w.intersection(a).contains(x)
            v.contains(x) and a.contains(x)
            w.contains(x) and a.contains(x)
            v.contains(x) and w.contains(x)
            v.intersection(w).contains(x)
            v.intersection(w).contains(x) and a.contains(x)
            r.contains(x)
        }
        if r.contains(x) {
            v.intersection(w).contains(x) and a.contains(x)
            v.contains(x) and w.contains(x)
            v.intersection(a).contains(x)
            w.intersection(a).contains(x)
            l.contains(x)
        }
        l.contains(x) = r.contains(x)
    }
    set_ext(l, r)
}

/// Helper: the union of two trace sets is the trace of the ambient union.
theorem subspace_union_distrib[T](v: Set[T], w: Set[T], a: Set[T]) {
    v.intersection(a).union(w.intersection(a)) = v.union(w).intersection(a)
} by {
    let l = v.intersection(a).union(w.intersection(a))
    let r = v.union(w).intersection(a)
    forall(x: T) {
        union_contains_eq(v.intersection(a), w.intersection(a), x)
        intersection_contains_eq(v, a, x)
        intersection_contains_eq(w, a, x)
        union_contains_eq(v, w, x)
        intersection_contains_eq(v.union(w), a, x)
        if l.contains(x) {
            if v.intersection(a).contains(x) {
                v.contains(x) and a.contains(x)
                v.union(w).contains(x)
                r.contains(x)
            } else {
                w.intersection(a).contains(x)
                w.contains(x) and a.contains(x)
                v.union(w).contains(x)
                r.contains(x)
            }
        }
        if r.contains(x) {
            intersection_contains_eq(v.union(w), a, x)
            v.union(w).contains(x)
            a.contains(x)
            union_contains_eq(v, w, x)
            if v.contains(x) {
                v.intersection(a).contains(x)
                l.contains(x)
            } else {
                w.contains(x)
                w.intersection(a).contains(x)
                l.contains(x)
            }
        }
        l.contains(x) = r.contains(x)
    }
    set_ext(l, r)
}

/// Helper: intersecting a trace set with an ambient set is the trace of the ambient intersection.
theorem subspace_inter_ambient_distrib[T](v: Set[T], w: Set[T], a: Set[T]) {
    v.intersection(a).intersection(w) = v.intersection(w).intersection(a)
} by {
    let l = v.intersection(a).intersection(w)
    let r = v.intersection(w).intersection(a)
    forall(x: T) {
        intersection_contains_eq(v.intersection(a), w, x)
        intersection_contains_eq(v, a, x)
        intersection_contains_eq(v, w, x)
        intersection_contains_eq(v.intersection(w), a, x)
        if l.contains(x) {
            intersection_contains_eq(v.intersection(a), w, x)
            v.intersection(a).contains(x)
            w.contains(x)
            intersection_contains_eq(v, a, x)
            v.contains(x)
            a.contains(x)
            intersection_contains_eq(v, w, x)
            v.intersection(w).contains(x)
            intersection_contains_eq(v.intersection(w), a, x)
            r.contains(x)
        }
        if r.contains(x) {
            intersection_contains_eq(v.intersection(w), a, x)
            v.intersection(w).contains(x)
            a.contains(x)
            intersection_contains_eq(v, w, x)
            v.contains(x)
            w.contains(x)
            intersection_contains_eq(v, a, x)
            v.intersection(a).contains(x)
            intersection_contains_eq(v.intersection(a), w, x)
            l.contains(x)
        }
        l.contains(x) = r.contains(x)
    }
    set_ext(l, r)
}

/// Helper: the carrier-relative complement of an ambient trace is the trace of the ambient complement.
theorem subspace_relative_complement_trace[T](a: Set[T], v: Set[T]) {
    a.difference(v.intersection(a)) = v.c.intersection(a)
} by {
    let l = a.difference(v.intersection(a))
    let r = v.c.intersection(a)
    forall(x: T) {
        difference_contains_eq(a, v.intersection(a), x)
        intersection_contains_eq(v, a, x)
        compl_contains_eq(v, x)
        intersection_contains_eq(v.c, a, x)
        if l.contains(x) {
            difference_contains_eq(a, v.intersection(a), x)
            a.contains(x)
            not v.intersection(a).contains(x)
            if v.contains(x) {
                intersection_contains_eq(v, a, x)
                v.intersection(a).contains(x)
                false
            }
            not v.contains(x)
            compl_contains_eq(v, x)
            v.c.contains(x)
            intersection_contains_eq(v.c, a, x)
            r.contains(x)
        }
        if r.contains(x) {
            intersection_contains_eq(v.c, a, x)
            v.c.contains(x)
            a.contains(x)
            compl_contains_eq(v, x)
            not v.contains(x)
            if v.intersection(a).contains(x) {
                intersection_contains_eq(v, a, x)
                v.contains(x)
                false
            }
            not v.intersection(a).contains(x)
            difference_contains_eq(a, v.intersection(a), x)
            l.contains(x)
        }
        l.contains(x) = r.contains(x)
    }
    set_ext(l, r)
}

/// The intersection of two subspace-open sets is subspace-open.
theorem subspace_open_inter[T: TopologicalSpace](a: Set[T], u: Set[T], w: Set[T]) {
    is_open_in_subspace[T](a, u) and is_open_in_subspace[T](a, w)
        implies is_open_in_subspace[T](a, u.intersection(w))
} by {
    if is_open_in_subspace[T](a, u) and is_open_in_subspace[T](a, w) {
        let v: Set[T] satisfy {
            T.is_open(v) and u = v.intersection(a)
        }
        let v2: Set[T] satisfy {
            T.is_open(v2) and w = v2.intersection(a)
        }
        open_inter[T](v, v2)
        T.is_open(v.intersection(v2))
        subspace_inter_distrib[T](v, v2, a)
        v.intersection(a).intersection(v2.intersection(a)) = v.intersection(v2).intersection(a)
        u.intersection(w) = v.intersection(a).intersection(v2.intersection(a))
        u.intersection(w) = v.intersection(v2).intersection(a)
        T.is_open(v.intersection(v2)) and u.intersection(w) = v.intersection(v2).intersection(a)
        exists(v3: Set[T]) {
            T.is_open(v3) and u.intersection(w) = v3.intersection(a)
        }
    }
}

/// The intersection of a subspace `a` with any ambient open is a subspace-open set.
theorem subspace_open_from_ambient[T: TopologicalSpace](a: Set[T], v: Set[T]) {
    T.is_open(v) implies is_open_in_subspace[T](a, v.intersection(a))
} by {
    if T.is_open(v) {
        T.is_open(v) and v.intersection(a) = v.intersection(a)
        exists(w: Set[T]) {
            T.is_open(w) and v.intersection(a) = w.intersection(a)
        }
    }
}

/// The union of two subspace-open sets is subspace-open.
theorem subspace_open_union[T: TopologicalSpace](a: Set[T], u: Set[T], w: Set[T]) {
    is_open_in_subspace[T](a, u) and is_open_in_subspace[T](a, w)
        implies is_open_in_subspace[T](a, u.union(w))
} by {
    if is_open_in_subspace[T](a, u) and is_open_in_subspace[T](a, w) {
        let v: Set[T] satisfy {
            T.is_open(v) and u = v.intersection(a)
        }
        let v2: Set[T] satisfy {
            T.is_open(v2) and w = v2.intersection(a)
        }
        open_union_open[T](v, v2)
        T.is_open(v.union(v2))
        subspace_union_distrib[T](v, v2, a)
        v.intersection(a).union(v2.intersection(a)) = v.union(v2).intersection(a)
        u.union(w) = v.intersection(a).union(v2.intersection(a))
        u.union(w) = v.union(v2).intersection(a)
        T.is_open(v.union(v2)) and u.union(w) = v.union(v2).intersection(a)
        exists(v3: Set[T]) {
            T.is_open(v3) and u.union(w) = v3.intersection(a)
        }
    }
}

/// Every subspace-open set is contained in its carrier.
theorem subspace_open_subset_carrier[T: TopologicalSpace](a: Set[T], u: Set[T]) {
    is_open_in_subspace[T](a, u) implies u.subset(a)
} by {
    if is_open_in_subspace[T](a, u) {
        let v: Set[T] satisfy {
            T.is_open(v) and u = v.intersection(a)
        }
        sets_subset_intersection(v, a)
        v.intersection(a).subset(a)
        u.subset(a)
    }
}

/// Intersecting a subspace-open set with an ambient open set is subspace-open.
theorem subspace_open_inter_ambient[T: TopologicalSpace](a: Set[T], u: Set[T], v: Set[T]) {
    is_open_in_subspace[T](a, u) and T.is_open(v)
        implies is_open_in_subspace[T](a, u.intersection(v))
} by {
    if is_open_in_subspace[T](a, u) and T.is_open(v) {
        let w: Set[T] satisfy {
            T.is_open(w) and u = w.intersection(a)
        }
        open_inter[T](w, v)
        T.is_open(w.intersection(v))
        subspace_inter_ambient_distrib[T](w, v, a)
        w.intersection(a).intersection(v) = w.intersection(v).intersection(a)
        u.intersection(v) = w.intersection(a).intersection(v)
        u.intersection(v) = w.intersection(v).intersection(a)
        T.is_open(w.intersection(v)) and u.intersection(v) = w.intersection(v).intersection(a)
        exists(v2: Set[T]) {
            T.is_open(v2) and u.intersection(v) = v2.intersection(a)
        }
    }
}

/// Uniting a subspace-open set with an ambient-open trace is subspace-open.
theorem subspace_open_union_ambient[T: TopologicalSpace](a: Set[T], u: Set[T], v: Set[T]) {
    is_open_in_subspace[T](a, u) and T.is_open(v)
        implies is_open_in_subspace[T](a, u.union(v.intersection(a)))
} by {
    if is_open_in_subspace[T](a, u) and T.is_open(v) {
        let w: Set[T] satisfy {
            T.is_open(w) and u = w.intersection(a)
        }
        open_union_open[T](w, v)
        T.is_open(w.union(v))
        subspace_union_distrib[T](w, v, a)
        w.intersection(a).union(v.intersection(a)) = w.union(v).intersection(a)
        u.union(v.intersection(a)) = w.intersection(a).union(v.intersection(a))
        u.union(v.intersection(a)) = w.union(v).intersection(a)
        T.is_open(w.union(v)) and u.union(v.intersection(a)) = w.union(v).intersection(a)
        exists(v2: Set[T]) {
            T.is_open(v2) and u.union(v.intersection(a)) = v2.intersection(a)
        }
    }
}

/// True if `u` is closed in the subspace `a`: it is the intersection of an ambient closed set
/// with `a`.
define is_closed_in_subspace[T: TopologicalSpace](a: Set[T], u: Set[T]) -> Bool {
    exists(c: Set[T]) {
        is_closed[T](c) and u = c.intersection(a)
    }
}

/// The empty set is closed in any subspace.
theorem subspace_closed_empty[T: TopologicalSpace](a: Set[T]) {
    is_closed_in_subspace[T](a, Set[T].empty_set)
} by {
    let e = Set[T].empty_set
    closed_empty[T]
    is_closed[T](e)
    forall(x: T) {
        intersection_contains_eq(e, a, x)
        empty_set_contains_eq[T](x)
        e.intersection(a).contains(x) = e.contains(x)
    }
    set_ext(e.intersection(a), e)
    e.intersection(a) = e
    e = e.intersection(a)
    is_closed[T](e) and e = e.intersection(a)
    exists(c: Set[T]) {
        is_closed[T](c) and e = c.intersection(a)
    }
}

/// The whole subspace `a` is closed in itself.
theorem subspace_closed_self[T: TopologicalSpace](a: Set[T]) {
    is_closed_in_subspace[T](a, a)
} by {
    closed_universal[T]
    is_closed[T](Set[T].universal_set)
    forall(x: T) {
        intersection_contains_eq(Set[T].universal_set, a, x)
        universal_set_contains_eq[T](x)
        Set[T].universal_set.intersection(a).contains(x) = a.contains(x)
    }
    set_ext(Set[T].universal_set.intersection(a), a)
    Set[T].universal_set.intersection(a) = a
    a = Set[T].universal_set.intersection(a)
    is_closed[T](Set[T].universal_set) and a = Set[T].universal_set.intersection(a)
    exists(c: Set[T]) {
        is_closed[T](c) and a = c.intersection(a)
    }
}

/// The intersection of two subspace-closed sets is subspace-closed.
theorem subspace_closed_inter[T: TopologicalSpace](a: Set[T], u: Set[T], w: Set[T]) {
    is_closed_in_subspace[T](a, u) and is_closed_in_subspace[T](a, w)
        implies is_closed_in_subspace[T](a, u.intersection(w))
} by {
    if is_closed_in_subspace[T](a, u) and is_closed_in_subspace[T](a, w) {
        let c: Set[T] satisfy {
            is_closed[T](c) and u = c.intersection(a)
        }
        let c2: Set[T] satisfy {
            is_closed[T](c2) and w = c2.intersection(a)
        }
        closed_inter[T](c, c2)
        is_closed[T](c.intersection(c2))
        subspace_inter_distrib[T](c, c2, a)
        c.intersection(a).intersection(c2.intersection(a)) = c.intersection(c2).intersection(a)
        u.intersection(w) = c.intersection(a).intersection(c2.intersection(a))
        u.intersection(w) = c.intersection(c2).intersection(a)
        is_closed[T](c.intersection(c2)) and u.intersection(w) = c.intersection(c2).intersection(a)
        exists(c3: Set[T]) {
            is_closed[T](c3) and u.intersection(w) = c3.intersection(a)
        }
    }
}

/// The intersection of a subspace `a` with any ambient closed set is subspace-closed.
theorem subspace_closed_from_ambient[T: TopologicalSpace](a: Set[T], c: Set[T]) {
    is_closed[T](c) implies is_closed_in_subspace[T](a, c.intersection(a))
} by {
    if is_closed[T](c) {
        is_closed[T](c) and c.intersection(a) = c.intersection(a)
        exists(c2: Set[T]) {
            is_closed[T](c2) and c.intersection(a) = c2.intersection(a)
        }
    }
}

/// The union of two subspace-closed sets is subspace-closed.
theorem subspace_closed_union[T: TopologicalSpace](a: Set[T], u: Set[T], w: Set[T]) {
    is_closed_in_subspace[T](a, u) and is_closed_in_subspace[T](a, w)
        implies is_closed_in_subspace[T](a, u.union(w))
} by {
    if is_closed_in_subspace[T](a, u) and is_closed_in_subspace[T](a, w) {
        let c: Set[T] satisfy {
            is_closed[T](c) and u = c.intersection(a)
        }
        let c2: Set[T] satisfy {
            is_closed[T](c2) and w = c2.intersection(a)
        }
        closed_union_closed[T](c, c2)
        is_closed[T](c.union(c2))
        subspace_union_distrib[T](c, c2, a)
        c.intersection(a).union(c2.intersection(a)) = c.union(c2).intersection(a)
        u.union(w) = c.intersection(a).union(c2.intersection(a))
        u.union(w) = c.union(c2).intersection(a)
        is_closed[T](c.union(c2)) and u.union(w) = c.union(c2).intersection(a)
        exists(c3: Set[T]) {
            is_closed[T](c3) and u.union(w) = c3.intersection(a)
        }
    }
}

/// Every subspace-closed set is contained in its carrier.
theorem subspace_closed_subset_carrier[T: TopologicalSpace](a: Set[T], u: Set[T]) {
    is_closed_in_subspace[T](a, u) implies u.subset(a)
} by {
    if is_closed_in_subspace[T](a, u) {
        let c: Set[T] satisfy {
            is_closed[T](c) and u = c.intersection(a)
        }
        sets_subset_intersection(c, a)
        c.intersection(a).subset(a)
        u.subset(a)
    }
}

/// Intersecting a subspace-closed set with an ambient closed set is subspace-closed.
theorem subspace_closed_inter_ambient[T: TopologicalSpace](a: Set[T], u: Set[T], c: Set[T]) {
    is_closed_in_subspace[T](a, u) and is_closed[T](c)
        implies is_closed_in_subspace[T](a, u.intersection(c))
} by {
    if is_closed_in_subspace[T](a, u) and is_closed[T](c) {
        let d: Set[T] satisfy {
            is_closed[T](d) and u = d.intersection(a)
        }
        closed_inter[T](d, c)
        is_closed[T](d.intersection(c))
        subspace_inter_ambient_distrib[T](d, c, a)
        d.intersection(a).intersection(c) = d.intersection(c).intersection(a)
        u.intersection(c) = d.intersection(a).intersection(c)
        u.intersection(c) = d.intersection(c).intersection(a)
        is_closed[T](d.intersection(c)) and u.intersection(c) = d.intersection(c).intersection(a)
        exists(c2: Set[T]) {
            is_closed[T](c2) and u.intersection(c) = c2.intersection(a)
        }
    }
}

/// Uniting a subspace-closed set with an ambient-closed trace is subspace-closed.
theorem subspace_closed_union_ambient[T: TopologicalSpace](a: Set[T], u: Set[T], c: Set[T]) {
    is_closed_in_subspace[T](a, u) and is_closed[T](c)
        implies is_closed_in_subspace[T](a, u.union(c.intersection(a)))
} by {
    if is_closed_in_subspace[T](a, u) and is_closed[T](c) {
        let d: Set[T] satisfy {
            is_closed[T](d) and u = d.intersection(a)
        }
        closed_union_closed[T](d, c)
        is_closed[T](d.union(c))
        subspace_union_distrib[T](d, c, a)
        d.intersection(a).union(c.intersection(a)) = d.union(c).intersection(a)
        u.union(c.intersection(a)) = d.intersection(a).union(c.intersection(a))
        u.union(c.intersection(a)) = d.union(c).intersection(a)
        is_closed[T](d.union(c)) and u.union(c.intersection(a)) = d.union(c).intersection(a)
        exists(c2: Set[T]) {
            is_closed[T](c2) and u.union(c.intersection(a)) = c2.intersection(a)
        }
    }
}

/// The carrier-relative complement of a subspace-open set is subspace-closed.
theorem subspace_open_relative_complement_closed[T: TopologicalSpace](a: Set[T], u: Set[T]) {
    is_open_in_subspace[T](a, u) implies is_closed_in_subspace[T](a, a.difference(u))
} by {
    if is_open_in_subspace[T](a, u) {
        let v: Set[T] satisfy {
            T.is_open(v) and u = v.intersection(a)
        }
        subspace_relative_complement_trace[T](a, v)
        a.difference(v.intersection(a)) = v.c.intersection(a)
        a.difference(u) = a.difference(v.intersection(a))
        a.difference(u) = v.c.intersection(a)
        is_closed[T](v.c) = T.is_open(v.c.c)
        compl_of_compl_is_self(v)
        v.c.c = v
        T.is_open(v.c.c)
        is_closed[T](v.c)
        is_closed[T](v.c) and a.difference(u) = v.c.intersection(a)
        exists(c: Set[T]) {
            is_closed[T](c) and a.difference(u) = c.intersection(a)
        }
    }
}

/// The carrier-relative complement of a subspace-closed set is subspace-open.
theorem subspace_closed_relative_complement_open[T: TopologicalSpace](a: Set[T], u: Set[T]) {
    is_closed_in_subspace[T](a, u) implies is_open_in_subspace[T](a, a.difference(u))
} by {
    if is_closed_in_subspace[T](a, u) {
        let c: Set[T] satisfy {
            is_closed[T](c) and u = c.intersection(a)
        }
        subspace_relative_complement_trace[T](a, c)
        a.difference(c.intersection(a)) = c.c.intersection(a)
        a.difference(u) = a.difference(c.intersection(a))
        a.difference(u) = c.c.intersection(a)
        is_closed[T](c) = T.is_open(c.c)
        T.is_open(c.c)
        T.is_open(c.c) and a.difference(u) = c.c.intersection(a)
        exists(v: Set[T]) {
            T.is_open(v) and a.difference(u) = v.intersection(a)
        }
    }
}

/// Carrier type for the subspace topology on `a ⊆ T`.
structure Subspace[T: TopologicalSpace, a: Set[T]] {
    /// The underlying ambient point.
    value: T
} constraint {
    a.contains(value)
}

/// True if the ambient open `v` traces onto the subspace-set `u`.
define has_subspace_witness[T: TopologicalSpace](
    a: Set[T], u: Set[Subspace[T, a]], v: Set[T]
) -> Bool {
    T.is_open(v) and forall(p: Subspace[T, a]) {
        u.contains(p) = v.contains(p.value)
    }
}

/// True if `u` is open in the subspace topology on `a ⊆ T`: it is the trace, on the
/// subspace, of some ambient open set.
define subspace_open[T: TopologicalSpace](a: Set[T], u: Set[Subspace[T, a]]) -> Bool {
    exists(v: Set[T]) {
        has_subspace_witness(a, u, v)
    }
}

/// Helper: the empty set is subspace-open. Witness: ambient empty set.
theorem subspace_open_empty_helper[T: TopologicalSpace](a: Set[T]) {
    subspace_open(a, Set[Subspace[T, a]].empty_set)
} by {
    let e = Set[T].empty_set
    let ee = Set[Subspace[T, a]].empty_set
    open_empty[T]
    T.is_open(e)
    forall(p: Subspace[T, a]) {
        empty_set_contains_eq[Subspace[T, a]](p)
        empty_set_contains_eq[T](p.value)
        ee.contains(p) = e.contains(p.value)
    }
    T.is_open(e) and forall(p: Subspace[T, a]) {
        ee.contains(p) = e.contains(p.value)
    }
    has_subspace_witness(a, ee, e)
    subspace_open(a, ee) = exists(v: Set[T]) {
        has_subspace_witness(a, ee, v)
    }
    exists(v: Set[T]) {
        T.is_open(v) and forall(p: Subspace[T, a]) {
            ee.contains(p) = v.contains(p.value)
        }
    }
    subspace_open(a, ee)
}

/// Helper: the universal set is subspace-open. Witness: ambient universal set.
theorem subspace_open_universal_helper[T: TopologicalSpace](a: Set[T]) {
    subspace_open(a, Set[Subspace[T, a]].universal_set)
} by {
    let u = Set[T].universal_set
    let uu = Set[Subspace[T, a]].universal_set
    open_universal[T]
    T.is_open(u)
    forall(p: Subspace[T, a]) {
        universal_set_contains_eq[Subspace[T, a]](p)
        universal_set_contains_eq[T](p.value)
        uu.contains(p) = u.contains(p.value)
    }
    T.is_open(u) and forall(p: Subspace[T, a]) {
        uu.contains(p) = u.contains(p.value)
    }
    has_subspace_witness(a, uu, u)
    subspace_open(a, uu) = exists(v: Set[T]) {
        has_subspace_witness(a, uu, v)
    }
    exists(v: Set[T]) {
        T.is_open(v) and forall(p: Subspace[T, a]) {
            uu.contains(p) = v.contains(p.value)
        }
    }
    subspace_open(a, uu)
}

/// Helper: subspace_open is closed under binary intersection.
theorem subspace_open_inter_helper[T: TopologicalSpace](
    a: Set[T], u: Set[Subspace[T, a]], w: Set[Subspace[T, a]]
) {
    subspace_open(a, u) and subspace_open(a, w) implies subspace_open(a, u.intersection(w))
} by {
    if subspace_open(a, u) and subspace_open(a, w) {
        exists(v1: Set[T]) {
            has_subspace_witness(a, u, v1)
        }
        let v1: Set[T] satisfy {
            has_subspace_witness(a, u, v1)
        }
        exists(v2: Set[T]) {
            has_subspace_witness(a, w, v2)
        }
        let v2: Set[T] satisfy {
            has_subspace_witness(a, w, v2)
        }
        let v3 = v1.intersection(v2)
        T.is_open(v1) and forall(p: Subspace[T, a]) {
            u.contains(p) = v1.contains(p.value)
        }
        T.is_open(v2) and forall(p: Subspace[T, a]) {
            w.contains(p) = v2.contains(p.value)
        }
        open_inter[T](v1, v2)
        T.is_open(v3)
        forall(p: Subspace[T, a]) {
            intersection_contains_eq(u, w, p)
            intersection_contains_eq(v1, v2, p.value)
            u.contains(p) = v1.contains(p.value)
            w.contains(p) = v2.contains(p.value)
            u.intersection(w).contains(p) = v3.contains(p.value)
        }
        has_subspace_witness(a, u.intersection(w), v3)
    }
}

/// Family of ambient witnesses for a family of subspace-opens: ambient opens
/// `v` whose trace coincides with some member of `fam`.
define subspace_witness_family[T: TopologicalSpace](
    a: Set[T], fam: Set[Subspace[T, a]] -> Bool, v: Set[T]
) -> Bool {
    T.is_open(v) and exists(s: Set[Subspace[T, a]]) {
        fam(s) and forall(p: Subspace[T, a]) {
            s.contains(p) = v.contains(p.value)
        }
    }
}

/// Helper: subspace_open is closed under big union.
theorem subspace_open_big_union_helper[T: TopologicalSpace](
    a: Set[T], fam: Set[Subspace[T, a]] -> Bool
) {
    (forall(s: Set[Subspace[T, a]]) {
        fam(s) implies subspace_open(a, s)
    }) implies subspace_open(a, big_union(fam))
} by {
    if forall(s: Set[Subspace[T, a]]) { fam(s) implies subspace_open(a, s) } {
        let q = subspace_witness_family(a, fam)
        forall(v: Set[T]) {
            if q(v) {
                T.is_open(v)
            }
        }
        open_big_union[T](q)
        let big_v = big_union(q)
        T.is_open(big_v)
        forall(p: Subspace[T, a]) {
            big_union_contains_eq(fam, p)
            big_union_contains_eq(q, p.value)
            if big_union(fam).contains(p) {
                exists(s: Set[Subspace[T, a]]) {
                    fam(s) and s.contains(p)
                }
                let s: Set[Subspace[T, a]] satisfy {
                    fam(s) and s.contains(p)
                }
                fam(s) implies subspace_open(a, s)
                exists(vv: Set[T]) {
                    has_subspace_witness(a, s, vv)
                }
                let v: Set[T] satisfy {
                    has_subspace_witness(a, s, v)
                }
                T.is_open(v) and forall(pp: Subspace[T, a]) {
                    s.contains(pp) = v.contains(pp.value)
                }
                s.contains(p) = v.contains(p.value)
                v.contains(p.value)
                fam(s) and forall(pp: Subspace[T, a]) {
                    s.contains(pp) = v.contains(pp.value)
                }
                exists(ss: Set[Subspace[T, a]]) {
                    fam(ss) and forall(pp: Subspace[T, a]) {
                        ss.contains(pp) = v.contains(pp.value)
                    }
                }
                q(v)
                q(v) and v.contains(p.value)
                exists(vv: Set[T]) {
                    q(vv) and vv.contains(p.value)
                }
                big_v.contains(p.value)
            }
            if big_v.contains(p.value) {
                exists(v: Set[T]) {
                    q(v) and v.contains(p.value)
                }
                let v: Set[T] satisfy {
                    q(v) and v.contains(p.value)
                }
                q(v)
                exists(s: Set[Subspace[T, a]]) {
                    fam(s) and forall(pp: Subspace[T, a]) {
                        s.contains(pp) = v.contains(pp.value)
                    }
                }
                let s: Set[Subspace[T, a]] satisfy {
                    fam(s) and forall(pp: Subspace[T, a]) {
                        s.contains(pp) = v.contains(pp.value)
                    }
                }
                s.contains(p) = v.contains(p.value)
                s.contains(p)
                exists(ss: Set[Subspace[T, a]]) {
                    fam(ss) and ss.contains(p)
                }
                big_union(fam).contains(p)
            }
            big_union(fam).contains(p) = big_v.contains(p.value)
        }
        T.is_open(big_v) and forall(p: Subspace[T, a]) {
            big_union(fam).contains(p) = big_v.contains(p.value)
        }
        has_subspace_witness(a, big_union(fam), big_v)
    }
}

instance Subspace[T: TopologicalSpace, a: Set[T]]: TopologicalSpace {
    let is_open: Set[Subspace[T, a]] -> Bool = subspace_open(a)
}


/// Membership predicate for the product box of two sets.
define box_contains[X, Y](u: Set[X], v: Set[Y], p: Pair[X, Y]) -> Bool {
    u.contains(p.first) and v.contains(p.second)
}

/// The product box of two sets: pairs whose components lie in the corresponding factor.
define box[X, Y](u: Set[X], v: Set[Y]) -> Set[Pair[X, Y]] {
    Set[Pair[X, Y]].new(box_contains(u, v))
}

/// Membership in a product box is componentwise membership.
theorem box_contains_eq[X, Y](u: Set[X], v: Set[Y], p: Pair[X, Y]) {
    box(u, v).contains(p) = (u.contains(p.first) and v.contains(p.second))
} by {
    box(u, v).contains(p) = box_contains(u, v, p)
}

/// True if a subset of `Pair[X, Y]` is a basic open of the product topology: a box
/// of an open subset of X with an open subset of Y.
define is_box[X: TopologicalSpace, Y: TopologicalSpace](b: Set[Pair[X, Y]]) -> Bool {
    exists(u: Set[X], v: Set[Y]) {
        X.is_open(u) and Y.is_open(v) and b = box(u, v)
    }
}

/// The product universal set is a box.
theorem universal_pair_is_box[X: TopologicalSpace, Y: TopologicalSpace] {
    is_box[X, Y](Set[Pair[X, Y]].universal_set)
} by {
    open_universal[X]
    open_universal[Y]
    let bx = box(Set[X].universal_set, Set[Y].universal_set)
    forall(p: Pair[X, Y]) {
        box_contains_eq(Set[X].universal_set, Set[Y].universal_set, p)
        universal_set_contains_eq[X](p.first)
        universal_set_contains_eq[Y](p.second)
        universal_set_contains_eq[Pair[X, Y]](p)
        bx.contains(p) = Set[Pair[X, Y]].universal_set.contains(p)
    }
    set_ext(bx, Set[Pair[X, Y]].universal_set)
    bx = Set[Pair[X, Y]].universal_set
    Set[Pair[X, Y]].universal_set = box(Set[X].universal_set, Set[Y].universal_set)
    X.is_open(Set[X].universal_set) and Y.is_open(Set[Y].universal_set) and Set[Pair[X, Y]].universal_set = box(Set[X].universal_set, Set[Y].universal_set)
    exists(u: Set[X], v: Set[Y]) {
        X.is_open(u) and Y.is_open(v) and Set[Pair[X, Y]].universal_set = box(u, v)
    }
}

/// The intersection of two boxes is the box of componentwise intersections.
theorem box_intersection_eq[X, Y](u: Set[X], v: Set[Y], u2: Set[X], v2: Set[Y]) {
    box(u, v).intersection(box(u2, v2)) = box(u.intersection(u2), v.intersection(v2))
} by {
    let l = box(u, v).intersection(box(u2, v2))
    let r = box(u.intersection(u2), v.intersection(v2))
    forall(p: Pair[X, Y]) {
        intersection_contains_eq(box(u, v), box(u2, v2), p)
        box_contains_eq(u, v, p)
        box_contains_eq(u2, v2, p)
        box_contains_eq(u.intersection(u2), v.intersection(v2), p)
        intersection_contains_eq(u, u2, p.first)
        intersection_contains_eq(v, v2, p.second)
        if l.contains(p) {
            box(u, v).contains(p) and box(u2, v2).contains(p)
            u.contains(p.first) and v.contains(p.second)
            u2.contains(p.first) and v2.contains(p.second)
            u.contains(p.first) and u2.contains(p.first)
            v.contains(p.second) and v2.contains(p.second)
            u.intersection(u2).contains(p.first)
            v.intersection(v2).contains(p.second)
            r.contains(p)
        }
        if r.contains(p) {
            u.intersection(u2).contains(p.first) and v.intersection(v2).contains(p.second)
            u.contains(p.first) and u2.contains(p.first)
            v.contains(p.second) and v2.contains(p.second)
            u.contains(p.first) and v.contains(p.second)
            u2.contains(p.first) and v2.contains(p.second)
            box(u, v).contains(p)
            box(u2, v2).contains(p)
            l.contains(p)
        }
        l.contains(p) = r.contains(p)
    }
    set_ext(l, r)
}

/// The intersection of two product boxes is itself a product box.
theorem box_inter_is_box[X: TopologicalSpace, Y: TopologicalSpace](
    b1: Set[Pair[X, Y]], b2: Set[Pair[X, Y]]) {
    is_box[X, Y](b1) and is_box[X, Y](b2) implies is_box[X, Y](b1.intersection(b2))
} by {
    if is_box[X, Y](b1) and is_box[X, Y](b2) {
        is_box[X, Y](b1) = exists(u: Set[X], v: Set[Y]) {
            X.is_open(u) and Y.is_open(v) and b1 = box(u, v)
        }
        let u: Set[X] satisfy {
            exists(v: Set[Y]) {
                X.is_open(u) and Y.is_open(v) and b1 = box(u, v)
            }
        }
        let v: Set[Y] satisfy {
            X.is_open(u) and Y.is_open(v) and b1 = box(u, v)
        }
        is_box[X, Y](b2) = exists(uu: Set[X], vv: Set[Y]) {
            X.is_open(uu) and Y.is_open(vv) and b2 = box(uu, vv)
        }
        let u2: Set[X] satisfy {
            exists(vv: Set[Y]) {
                X.is_open(u2) and Y.is_open(vv) and b2 = box(u2, vv)
            }
        }
        let v2: Set[Y] satisfy {
            X.is_open(u2) and Y.is_open(v2) and b2 = box(u2, v2)
        }
        open_inter[X](u, u2)
        X.is_open(u.intersection(u2))
        open_inter[Y](v, v2)
        Y.is_open(v.intersection(v2))
        box_intersection_eq(u, v, u2, v2)
        box(u, v).intersection(box(u2, v2)) = box(u.intersection(u2), v.intersection(v2))
        b1.intersection(b2) = box(u, v).intersection(box(u2, v2))
        b1.intersection(b2) = box(u.intersection(u2), v.intersection(v2))
        X.is_open(u.intersection(u2)) and Y.is_open(v.intersection(v2)) and b1.intersection(b2) = box(u.intersection(u2), v.intersection(v2))
        exists(uu: Set[X], vv: Set[Y]) {
            X.is_open(uu) and Y.is_open(vv) and b1.intersection(b2) = box(uu, vv)
        }
    }
}

/// The product box family covers `Pair[X, Y]`.
theorem is_box_covers[X: TopologicalSpace, Y: TopologicalSpace] {
    basis_covers(is_box[X, Y])
} by {
    universal_pair_is_box[X, Y]
    forall(p: Pair[X, Y]) {
        universal_set_contains_eq[Pair[X, Y]](p)
        Set[Pair[X, Y]].universal_set.contains(p)
        is_box[X, Y](Set[Pair[X, Y]].universal_set) and Set[Pair[X, Y]].universal_set.contains(p)
        exists(b: Set[Pair[X, Y]]) {
            is_box[X, Y](b) and b.contains(p)
        }
    }
}

/// Membership of a point in both members of a basis filter implies membership in the intersection.
theorem box_inter_is_box_with_point[X: TopologicalSpace, Y: TopologicalSpace](
    b1: Set[Pair[X, Y]], b2: Set[Pair[X, Y]], p: Pair[X, Y]) {
    is_box[X, Y](b1) and is_box[X, Y](b2) and b1.contains(p) and b2.contains(p)
        implies is_box[X, Y](b1.intersection(b2)) and b1.intersection(b2).contains(p)
} by {
    if is_box[X, Y](b1) and is_box[X, Y](b2) and b1.contains(p) and b2.contains(p) {
        box_inter_is_box[X, Y](b1, b2)
        intersection_contains_eq(b1, b2, p)
    }
}

/// Witness for the basis_filters property at a particular triple.
theorem is_box_filters_at[X: TopologicalSpace, Y: TopologicalSpace](
    b1: Set[Pair[X, Y]], b2: Set[Pair[X, Y]], p: Pair[X, Y]) {
    is_box[X, Y](b1) and is_box[X, Y](b2) and b1.contains(p) and b2.contains(p)
        implies exists(w: Set[Pair[X, Y]]) {
            is_box[X, Y](w) and w.contains(p) and w.subset(b1.intersection(b2))
        }
} by {
    if is_box[X, Y](b1) and is_box[X, Y](b2) and b1.contains(p) and b2.contains(p) {
        box_inter_is_box_with_point[X, Y](b1, b2, p)
        is_box[X, Y](b1.intersection(b2)) and b1.intersection(b2).contains(p)
        subset_refl(b1.intersection(b2))
        b1.intersection(b2).subset(b1.intersection(b2))
    }
}

/// The product box family is closed under refining intersections at common points.
theorem is_box_filters[X: TopologicalSpace, Y: TopologicalSpace] {
    basis_filters(is_box[X, Y])
} by {
    forall(b1: Set[Pair[X, Y]], b2: Set[Pair[X, Y]], p: Pair[X, Y]) {
        is_box_filters_at[X, Y](b1, b2, p)
    }
}

/// The product box family is a topological basis on `Pair[X, Y]`.
theorem is_box_is_topological_basis[X: TopologicalSpace, Y: TopologicalSpace] {
    is_topological_basis(is_box[X, Y])
} by {
    is_box_covers[X, Y]
    is_box_filters[X, Y]
    is_topological_basis(is_box[X, Y]) = (basis_covers(is_box[X, Y]) and basis_filters(is_box[X, Y]))
}

/// True if a subset of `Pair[X, Y]` is open in the product topology.
define is_open_in_product[X: TopologicalSpace, Y: TopologicalSpace](
    s: Set[Pair[X, Y]]) -> Bool {
    generated_open(is_box[X, Y], s)
}

/// Product opens are exactly the generated-open sets of the box family.
theorem is_open_in_product_iff[X: TopologicalSpace, Y: TopologicalSpace](
    s: Set[Pair[X, Y]]) {
    is_open_in_product[X, Y](s) = generated_open(is_box[X, Y], s)
}

/// Forward direction of `is_open_in_product_iff`: a product-open set is generated-open.
theorem is_open_in_product_imp_generated_open[X: TopologicalSpace, Y: TopologicalSpace](
    s: Set[Pair[X, Y]]) {
    is_open_in_product[X, Y](s) implies generated_open(is_box[X, Y], s)
} by {
    is_open_in_product_iff[X, Y](s)
}

/// Backward direction of `is_open_in_product_iff`: a generated-open set of the box
/// family is product-open.
theorem generated_open_imp_is_open_in_product[X: TopologicalSpace, Y: TopologicalSpace](
    s: Set[Pair[X, Y]]) {
    generated_open(is_box[X, Y], s) implies is_open_in_product[X, Y](s)
} by {
    is_open_in_product_iff[X, Y](s)
}

/// The empty set is open in the product topology.
theorem product_open_empty[X: TopologicalSpace, Y: TopologicalSpace] {
    is_open_in_product[X, Y](Set[Pair[X, Y]].empty_set)
} by {
    generated_open_empty[Pair[X, Y]](is_box[X, Y])
    is_open_in_product[X, Y](Set[Pair[X, Y]].empty_set) = generated_open(is_box[X, Y], Set[Pair[X, Y]].empty_set)
}

/// The whole space is open in the product topology.
theorem product_open_universal[X: TopologicalSpace, Y: TopologicalSpace] {
    is_open_in_product[X, Y](Set[Pair[X, Y]].universal_set)
} by {
    is_box_covers[X, Y]
    generated_open_universal[Pair[X, Y]](is_box[X, Y])
    is_open_in_product[X, Y](Set[Pair[X, Y]].universal_set) = generated_open(is_box[X, Y], Set[Pair[X, Y]].universal_set)
}

/// Membership predicate for the left preimage of a sum-set.
define sum_left_preimage_contains[T, U](w: Set[Sum[T, U]], x: T) -> Bool {
    w.contains(Sum.inl[T, U](x))
}

/// The left preimage of a sum-set: the set of `T` values whose left injection is in `w`.
define sum_left_preimage[T, U](w: Set[Sum[T, U]]) -> Set[T] {
    Set[T].new(sum_left_preimage_contains(w))
}

/// Membership predicate for the right preimage of a sum-set.
define sum_right_preimage_contains[T, U](w: Set[Sum[T, U]], y: U) -> Bool {
    w.contains(Sum.inr[T, U](y))
}

/// The right preimage of a sum-set: the set of `U` values whose right injection is in `w`.
define sum_right_preimage[T, U](w: Set[Sum[T, U]]) -> Set[U] {
    Set[U].new(sum_right_preimage_contains(w))
}

/// Membership in the left preimage is membership of the left injection in the sum-set.
theorem sum_left_preimage_contains_eq[T, U](w: Set[Sum[T, U]], x: T) {
    sum_left_preimage(w).contains(x) = w.contains(Sum.inl[T, U](x))
} by {
    sum_left_preimage(w).contains(x) = sum_left_preimage_contains(w, x)
}

/// Membership in the right preimage is membership of the right injection in the sum-set.
theorem sum_right_preimage_contains_eq[T, U](w: Set[Sum[T, U]], y: U) {
    sum_right_preimage(w).contains(y) = w.contains(Sum.inr[T, U](y))
} by {
    sum_right_preimage(w).contains(y) = sum_right_preimage_contains(w, y)
}

/// Left preimage of the empty set is empty.
theorem sum_left_preimage_empty[T, U] {
    sum_left_preimage[T, U](Set[Sum[T, U]].empty_set) = Set[T].empty_set
} by {
    forall(x: T) {
        sum_left_preimage_contains_eq[T, U](Set[Sum[T, U]].empty_set, x)
        empty_set_contains_eq[Sum[T, U]](Sum.inl[T, U](x))
        empty_set_contains_eq[T](x)
        sum_left_preimage[T, U](Set[Sum[T, U]].empty_set).contains(x) = Set[T].empty_set.contains(x)
    }
    set_ext(sum_left_preimage[T, U](Set[Sum[T, U]].empty_set), Set[T].empty_set)
}

/// Right preimage of the empty set is empty.
theorem sum_right_preimage_empty[T, U] {
    sum_right_preimage[T, U](Set[Sum[T, U]].empty_set) = Set[U].empty_set
} by {
    forall(y: U) {
        sum_right_preimage_contains_eq[T, U](Set[Sum[T, U]].empty_set, y)
        empty_set_contains_eq[Sum[T, U]](Sum.inr[T, U](y))
        empty_set_contains_eq[U](y)
        sum_right_preimage[T, U](Set[Sum[T, U]].empty_set).contains(y) = Set[U].empty_set.contains(y)
    }
    set_ext(sum_right_preimage[T, U](Set[Sum[T, U]].empty_set), Set[U].empty_set)
}

/// Left preimage of the universal set is universal.
theorem sum_left_preimage_universal[T, U] {
    sum_left_preimage[T, U](Set[Sum[T, U]].universal_set) = Set[T].universal_set
} by {
    forall(x: T) {
        sum_left_preimage_contains_eq[T, U](Set[Sum[T, U]].universal_set, x)
        universal_set_contains_eq[Sum[T, U]](Sum.inl[T, U](x))
        universal_set_contains_eq[T](x)
        sum_left_preimage[T, U](Set[Sum[T, U]].universal_set).contains(x) = Set[T].universal_set.contains(x)
    }
    set_ext(sum_left_preimage[T, U](Set[Sum[T, U]].universal_set), Set[T].universal_set)
}

/// Right preimage of the universal set is universal.
theorem sum_right_preimage_universal[T, U] {
    sum_right_preimage[T, U](Set[Sum[T, U]].universal_set) = Set[U].universal_set
} by {
    forall(y: U) {
        sum_right_preimage_contains_eq[T, U](Set[Sum[T, U]].universal_set, y)
        universal_set_contains_eq[Sum[T, U]](Sum.inr[T, U](y))
        universal_set_contains_eq[U](y)
        sum_right_preimage[T, U](Set[Sum[T, U]].universal_set).contains(y) = Set[U].universal_set.contains(y)
    }
    set_ext(sum_right_preimage[T, U](Set[Sum[T, U]].universal_set), Set[U].universal_set)
}

/// Left preimage commutes with binary intersection.
theorem sum_left_preimage_inter[T, U](w1: Set[Sum[T, U]], w2: Set[Sum[T, U]]) {
    sum_left_preimage(w1.intersection(w2))
        = sum_left_preimage(w1).intersection(sum_left_preimage(w2))
} by {
    forall(x: T) {
        sum_left_preimage_contains_eq(w1.intersection(w2), x)
        sum_left_preimage_contains_eq(w1, x)
        sum_left_preimage_contains_eq(w2, x)
        intersection_contains_eq(w1, w2, Sum.inl[T, U](x))
        intersection_contains_eq(sum_left_preimage(w1), sum_left_preimage(w2), x)
        sum_left_preimage(w1.intersection(w2)).contains(x) = sum_left_preimage(w1).intersection(sum_left_preimage(w2)).contains(x)
    }
    set_ext(sum_left_preimage(w1.intersection(w2)),
        sum_left_preimage(w1).intersection(sum_left_preimage(w2)))
}

/// Right preimage commutes with binary intersection.
theorem sum_right_preimage_inter[T, U](w1: Set[Sum[T, U]], w2: Set[Sum[T, U]]) {
    sum_right_preimage(w1.intersection(w2))
        = sum_right_preimage(w1).intersection(sum_right_preimage(w2))
} by {
    forall(y: U) {
        sum_right_preimage_contains_eq(w1.intersection(w2), y)
        sum_right_preimage_contains_eq(w1, y)
        sum_right_preimage_contains_eq(w2, y)
        intersection_contains_eq(w1, w2, Sum.inr[T, U](y))
        intersection_contains_eq(sum_right_preimage(w1), sum_right_preimage(w2), y)
        sum_right_preimage(w1.intersection(w2)).contains(y) = sum_right_preimage(w1).intersection(sum_right_preimage(w2)).contains(y)
    }
    set_ext(sum_right_preimage(w1.intersection(w2)),
        sum_right_preimage(w1).intersection(sum_right_preimage(w2)))
}

/// Membership predicate for the family of left preimages of a family of sum-sets.
define sum_left_preimage_family_contains[T, U](c: Set[Sum[T, U]] -> Bool, t: Set[T]) -> Bool {
    exists(s: Set[Sum[T, U]]) {
        c(s) and t = sum_left_preimage(s)
    }
}

/// The family of left preimages of a family of sum-sets.
define sum_left_preimage_family[T, U](c: Set[Sum[T, U]] -> Bool) -> (Set[T] -> Bool) {
    sum_left_preimage_family_contains(c)
}

/// Membership predicate for the family of right preimages of a family of sum-sets.
define sum_right_preimage_family_contains[T, U](c: Set[Sum[T, U]] -> Bool, t: Set[U]) -> Bool {
    exists(s: Set[Sum[T, U]]) {
        c(s) and t = sum_right_preimage(s)
    }
}

/// The family of right preimages of a family of sum-sets.
define sum_right_preimage_family[T, U](c: Set[Sum[T, U]] -> Bool) -> (Set[U] -> Bool) {
    sum_right_preimage_family_contains(c)
}

/// Membership in the left preimage family is being the left preimage of some member.
theorem sum_left_preimage_family_eq[T, U](c: Set[Sum[T, U]] -> Bool, t: Set[T]) {
    sum_left_preimage_family(c)(t) = exists(s: Set[Sum[T, U]]) {
        c(s) and t = sum_left_preimage(s)
    }
} by {
    sum_left_preimage_family(c)(t) = sum_left_preimage_family_contains(c, t)
}

/// Membership in the right preimage family is being the right preimage of some member.
theorem sum_right_preimage_family_eq[T, U](c: Set[Sum[T, U]] -> Bool, t: Set[U]) {
    sum_right_preimage_family(c)(t) = exists(s: Set[Sum[T, U]]) {
        c(s) and t = sum_right_preimage(s)
    }
} by {
    sum_right_preimage_family(c)(t) = sum_right_preimage_family_contains(c, t)
}

/// Left preimage commutes with big union.
theorem sum_left_preimage_big_union[T, U](c: Set[Sum[T, U]] -> Bool) {
    sum_left_preimage(big_union(c)) = big_union(sum_left_preimage_family(c))
} by {
    forall(x: T) {
        if sum_left_preimage(big_union(c)).contains(x) {
            sum_left_preimage_contains_eq(big_union(c), x)
            big_union(c).contains(Sum.inl[T, U](x))
            big_union_contains_eq(c, Sum.inl[T, U](x))
            let s: Set[Sum[T, U]] satisfy {
                c(s) and s.contains(Sum.inl[T, U](x))
            }
            sum_left_preimage_contains_eq(s, x)
            sum_left_preimage(s).contains(x)
            sum_left_preimage_family_eq(c, sum_left_preimage(s))
            sum_left_preimage_family(c)(sum_left_preimage(s))
            big_union_contains_of_member(sum_left_preimage_family(c), sum_left_preimage(s), x)
            big_union(sum_left_preimage_family(c)).contains(x)
        }
        if big_union(sum_left_preimage_family(c)).contains(x) {
            big_union_contains_eq(sum_left_preimage_family(c), x)
            let t: Set[T] satisfy {
                sum_left_preimage_family(c)(t) and t.contains(x)
            }
            sum_left_preimage_family_eq(c, t)
            let s: Set[Sum[T, U]] satisfy {
                c(s) and t = sum_left_preimage(s)
            }
            sum_left_preimage(s).contains(x)
            sum_left_preimage_contains_eq(s, x)
            s.contains(Sum.inl[T, U](x))
            big_union_contains_of_member(c, s, Sum.inl[T, U](x))
            big_union(c).contains(Sum.inl[T, U](x))
            sum_left_preimage_contains_eq(big_union(c), x)
            sum_left_preimage(big_union(c)).contains(x)
        }
        sum_left_preimage(big_union(c)).contains(x) = big_union(sum_left_preimage_family(c)).contains(x)
    }
    set_ext(sum_left_preimage(big_union(c)), big_union(sum_left_preimage_family(c)))
}

/// Right preimage commutes with big union.
theorem sum_right_preimage_big_union[T, U](c: Set[Sum[T, U]] -> Bool) {
    sum_right_preimage(big_union(c)) = big_union(sum_right_preimage_family(c))
} by {
    forall(y: U) {
        if sum_right_preimage(big_union(c)).contains(y) {
            sum_right_preimage_contains_eq(big_union(c), y)
            big_union(c).contains(Sum.inr[T, U](y))
            big_union_contains_eq(c, Sum.inr[T, U](y))
            let s: Set[Sum[T, U]] satisfy {
                c(s) and s.contains(Sum.inr[T, U](y))
            }
            sum_right_preimage_contains_eq(s, y)
            sum_right_preimage(s).contains(y)
            sum_right_preimage_family_eq(c, sum_right_preimage(s))
            sum_right_preimage_family(c)(sum_right_preimage(s))
            big_union_contains_of_member(sum_right_preimage_family(c), sum_right_preimage(s), y)
            big_union(sum_right_preimage_family(c)).contains(y)
        }
        if big_union(sum_right_preimage_family(c)).contains(y) {
            big_union_contains_eq(sum_right_preimage_family(c), y)
            let t: Set[U] satisfy {
                sum_right_preimage_family(c)(t) and t.contains(y)
            }
            sum_right_preimage_family_eq(c, t)
            let s: Set[Sum[T, U]] satisfy {
                c(s) and t = sum_right_preimage(s)
            }
            sum_right_preimage(s).contains(y)
            sum_right_preimage_contains_eq(s, y)
            s.contains(Sum.inr[T, U](y))
            big_union_contains_of_member(c, s, Sum.inr[T, U](y))
            big_union(c).contains(Sum.inr[T, U](y))
            sum_right_preimage_contains_eq(big_union(c), y)
            sum_right_preimage(big_union(c)).contains(y)
        }
        sum_right_preimage(big_union(c)).contains(y) = big_union(sum_right_preimage_family(c)).contains(y)
    }
    set_ext(sum_right_preimage(big_union(c)), big_union(sum_right_preimage_family(c)))
}

/// True if `w` is open in the sum topology on `Sum[X, Y]`: its left and right preimages
/// are open in their respective factors.
define is_open_in_sum[X: TopologicalSpace, Y: TopologicalSpace](w: Set[Sum[X, Y]]) -> Bool {
    X.is_open(sum_left_preimage(w)) and Y.is_open(sum_right_preimage(w))
}

/// The empty set is open in the sum topology.
theorem sum_open_empty[X: TopologicalSpace, Y: TopologicalSpace] {
    is_open_in_sum[X, Y](Set[Sum[X, Y]].empty_set)
} by {
    sum_left_preimage_empty[X, Y]
    sum_right_preimage_empty[X, Y]
    open_empty[X]
    open_empty[Y]
    X.is_open(sum_left_preimage[X, Y](Set[Sum[X, Y]].empty_set))
    Y.is_open(sum_right_preimage[X, Y](Set[Sum[X, Y]].empty_set))
}

/// The whole space is open in the sum topology.
theorem sum_open_universal[X: TopologicalSpace, Y: TopologicalSpace] {
    is_open_in_sum[X, Y](Set[Sum[X, Y]].universal_set)
} by {
    sum_left_preimage_universal[X, Y]
    sum_right_preimage_universal[X, Y]
    open_universal[X]
    open_universal[Y]
    X.is_open(sum_left_preimage[X, Y](Set[Sum[X, Y]].universal_set))
    Y.is_open(sum_right_preimage[X, Y](Set[Sum[X, Y]].universal_set))
}

/// Helper: the left part of sum_open_inter.
theorem sum_open_inter_left[X: TopologicalSpace, Y: TopologicalSpace](
    w1: Set[Sum[X, Y]], w2: Set[Sum[X, Y]]) {
    X.is_open(sum_left_preimage(w1)) and X.is_open(sum_left_preimage(w2))
        implies X.is_open(sum_left_preimage(w1.intersection(w2)))
} by {
    if X.is_open(sum_left_preimage(w1)) and X.is_open(sum_left_preimage(w2)) {
        open_inter[X](sum_left_preimage(w1), sum_left_preimage(w2))
        X.is_open(sum_left_preimage(w1).intersection(sum_left_preimage(w2)))
        sum_left_preimage_inter(w1, w2)
    }
}

/// Helper: the right part of sum_open_inter.
theorem sum_open_inter_right[X: TopologicalSpace, Y: TopologicalSpace](
    w1: Set[Sum[X, Y]], w2: Set[Sum[X, Y]]) {
    Y.is_open(sum_right_preimage(w1)) and Y.is_open(sum_right_preimage(w2))
        implies Y.is_open(sum_right_preimage(w1.intersection(w2)))
} by {
    if Y.is_open(sum_right_preimage(w1)) and Y.is_open(sum_right_preimage(w2)) {
        open_inter[Y](sum_right_preimage(w1), sum_right_preimage(w2))
        Y.is_open(sum_right_preimage(w1).intersection(sum_right_preimage(w2)))
        sum_right_preimage_inter(w1, w2)
    }
}

/// Sum-open membership unfolds to openness of the two preimages.
theorem is_open_in_sum_iff[X: TopologicalSpace, Y: TopologicalSpace](w: Set[Sum[X, Y]]) {
    is_open_in_sum[X, Y](w) = (X.is_open(sum_left_preimage(w)) and Y.is_open(sum_right_preimage(w)))
}

/// Building a sum-open from openness of both preimages.
theorem is_open_in_sum_intro[X: TopologicalSpace, Y: TopologicalSpace](w: Set[Sum[X, Y]]) {
    X.is_open(sum_left_preimage(w)) and Y.is_open(sum_right_preimage(w))
        implies is_open_in_sum[X, Y](w)
} by {
    is_open_in_sum_iff[X, Y](w)
}

/// Extracting the left preimage openness from a sum-open.
theorem is_open_in_sum_left[X: TopologicalSpace, Y: TopologicalSpace](w: Set[Sum[X, Y]]) {
    is_open_in_sum[X, Y](w) implies X.is_open(sum_left_preimage(w))
} by {
    is_open_in_sum_iff[X, Y](w)
}

/// Extracting the right preimage openness from a sum-open.
theorem is_open_in_sum_right[X: TopologicalSpace, Y: TopologicalSpace](w: Set[Sum[X, Y]]) {
    is_open_in_sum[X, Y](w) implies Y.is_open(sum_right_preimage(w))
} by {
    is_open_in_sum_iff[X, Y](w)
}

/// Paired extraction: from a sum-open we get openness of both preimages at once.
theorem is_open_in_sum_both[X: TopologicalSpace, Y: TopologicalSpace](w: Set[Sum[X, Y]]) {
    is_open_in_sum[X, Y](w) implies
        X.is_open(sum_left_preimage(w)) and Y.is_open(sum_right_preimage(w))
} by {
    is_open_in_sum_iff[X, Y](w)
}

/// The intersection of two sum-open sets is sum-open.
theorem sum_open_inter[X: TopologicalSpace, Y: TopologicalSpace](
    w1: Set[Sum[X, Y]], w2: Set[Sum[X, Y]]) {
    is_open_in_sum[X, Y](w1) and is_open_in_sum[X, Y](w2)
        implies is_open_in_sum[X, Y](w1.intersection(w2))
} by {
    if is_open_in_sum[X, Y](w1) and is_open_in_sum[X, Y](w2) {
        is_open_in_sum_both[X, Y](w1)
        is_open_in_sum_both[X, Y](w2)
        sum_open_inter_left[X, Y](w1, w2)
        sum_open_inter_right[X, Y](w1, w2)
        X.is_open(sum_left_preimage(w1.intersection(w2)))
        Y.is_open(sum_right_preimage(w1.intersection(w2)))
        X.is_open(sum_left_preimage(w1.intersection(w2))) and Y.is_open(sum_right_preimage(w1.intersection(w2)))
        is_open_in_sum_intro[X, Y](w1.intersection(w2))
        is_open_in_sum[X, Y](w1.intersection(w2))
    }
}

/// Helper: the left preimage of the big union of a family of sum-open sets is open.
theorem sum_open_big_union_left[X: TopologicalSpace, Y: TopologicalSpace](
    c: Set[Sum[X, Y]] -> Bool) {
    (forall(s: Set[Sum[X, Y]]) { c(s) implies is_open_in_sum[X, Y](s) })
        implies X.is_open(sum_left_preimage(big_union(c)))
} by {
    if forall(s: Set[Sum[X, Y]]) { c(s) implies is_open_in_sum[X, Y](s) } {
        forall(t: Set[X]) {
            if sum_left_preimage_family(c)(t) {
                sum_left_preimage_family_eq(c, t)
                let s: Set[Sum[X, Y]] satisfy {
                    c(s) and t = sum_left_preimage(s)
                }
                is_open_in_sum[X, Y](s)
                is_open_in_sum_left[X, Y](s)
                X.is_open(sum_left_preimage(s))
                X.is_open(t)
            }
        }
        open_big_union[X](sum_left_preimage_family(c))
        X.is_open(big_union(sum_left_preimage_family(c)))
        sum_left_preimage_big_union(c)
        X.is_open(sum_left_preimage(big_union(c)))
    }
}

/// Helper: the right preimage of the big union of a family of sum-open sets is open.
theorem sum_open_big_union_right[X: TopologicalSpace, Y: TopologicalSpace](
    c: Set[Sum[X, Y]] -> Bool) {
    (forall(s: Set[Sum[X, Y]]) { c(s) implies is_open_in_sum[X, Y](s) })
        implies Y.is_open(sum_right_preimage(big_union(c)))
} by {
    if forall(s: Set[Sum[X, Y]]) { c(s) implies is_open_in_sum[X, Y](s) } {
        forall(t: Set[Y]) {
            if sum_right_preimage_family(c)(t) {
                sum_right_preimage_family_eq(c, t)
                let s: Set[Sum[X, Y]] satisfy {
                    c(s) and t = sum_right_preimage(s)
                }
                is_open_in_sum[X, Y](s)
                is_open_in_sum_right[X, Y](s)
                Y.is_open(sum_right_preimage(s))
                Y.is_open(t)
            }
        }
        open_big_union[Y](sum_right_preimage_family(c))
        Y.is_open(big_union(sum_right_preimage_family(c)))
        sum_right_preimage_big_union(c)
        Y.is_open(sum_right_preimage(big_union(c)))
    }
}

/// The big union of a family of sum-open sets is sum-open.
theorem sum_open_big_union[X: TopologicalSpace, Y: TopologicalSpace](
    c: Set[Sum[X, Y]] -> Bool) {
    (forall(s: Set[Sum[X, Y]]) { c(s) implies is_open_in_sum[X, Y](s) })
        implies is_open_in_sum[X, Y](big_union(c))
} by {
    if forall(s: Set[Sum[X, Y]]) { c(s) implies is_open_in_sum[X, Y](s) } {
        sum_open_big_union_left[X, Y](c)
        X.is_open(sum_left_preimage(big_union(c)))
        sum_open_big_union_right[X, Y](c)
        Y.is_open(sum_right_preimage(big_union(c)))
        X.is_open(sum_left_preimage(big_union(c))) and Y.is_open(sum_right_preimage(big_union(c)))
        is_open_in_sum_intro[X, Y](big_union(c))
        is_open_in_sum[X, Y](big_union(c))
    }
}


/// True if `s` is open in the quotient topology associated with `r`: it is open in `T`
/// and saturated under `r` (it contains entire equivalence classes).
define is_quotient_open[T: TopologicalSpace](r: (T, T) -> Bool, s: Set[T]) -> Bool {
    T.is_open(s) and is_saturated_under(s, r)
}

/// The empty set is quotient-open: open and vacuously saturated.
theorem quotient_open_empty[T: TopologicalSpace](r: (T, T) -> Bool) {
    is_quotient_open[T](r, Set[T].empty_set)
} by {
    open_empty[T]
    T.is_open(Set[T].empty_set)
    forall(x: T, y: T) {
        if Set[T].empty_set.contains(x) and r(x, y) {
            empty_set_contains_eq[T](x)
            not Set[T].empty_set.contains(x)
            false
        }
    }
    is_saturated_under(Set[T].empty_set, r) = forall(x: T, y: T) {
        Set[T].empty_set.contains(x) and r(x, y) implies Set[T].empty_set.contains(y)
    }
}

/// The whole space is quotient-open: open and trivially saturated.
theorem quotient_open_universal[T: TopologicalSpace](r: (T, T) -> Bool) {
    is_quotient_open[T](r, Set[T].universal_set)
} by {
    open_universal[T]
    T.is_open(Set[T].universal_set)
    forall(x: T, y: T) {
        if Set[T].universal_set.contains(x) and r(x, y) {
            universal_set_contains_eq[T](y)
        }
    }
    is_saturated_under(Set[T].universal_set, r) = forall(x: T, y: T) {
        Set[T].universal_set.contains(x) and r(x, y) implies Set[T].universal_set.contains(y)
    }
}

/// The intersection of two quotient-open sets is quotient-open.
theorem quotient_open_inter[T: TopologicalSpace](r: (T, T) -> Bool, s: Set[T], t: Set[T]) {
    is_quotient_open[T](r, s) and is_quotient_open[T](r, t)
        implies is_quotient_open[T](r, s.intersection(t))
} by {
    if is_quotient_open[T](r, s) and is_quotient_open[T](r, t) {
        T.is_open(s) and T.is_open(t)
        open_inter[T](s, t)
        T.is_open(s.intersection(t))
        is_saturated_under(s, r)
        is_saturated_under(t, r)
        forall(x: T, y: T) {
            if s.intersection(t).contains(x) and r(x, y) {
                intersection_contains_eq(s, t, x)
                s.contains(x) and t.contains(x)
                is_saturated_under(s, r) = forall(x1: T, y1: T) {
                    s.contains(x1) and r(x1, y1) implies s.contains(y1)
                }
                s.contains(y)
                is_saturated_under(t, r) = forall(x1: T, y1: T) {
                    t.contains(x1) and r(x1, y1) implies t.contains(y1)
                }
                t.contains(y)
                intersection_contains_eq(s, t, y)
                s.intersection(t).contains(y)
            }
        }
        is_saturated_under(s.intersection(t), r) = forall(x: T, y: T) {
            s.intersection(t).contains(x) and r(x, y) implies s.intersection(t).contains(y)
        }
        is_saturated_under(s.intersection(t), r)
        T.is_open(s.intersection(t)) and is_saturated_under(s.intersection(t), r)
        is_quotient_open[T](r, s.intersection(t)) = (
            T.is_open(s.intersection(t)) and is_saturated_under(s.intersection(t), r)
        )
        is_quotient_open[T](r, s.intersection(t))
    }
}

/// The union of a family of quotient-open sets is quotient-open.
theorem quotient_open_big_union[T: TopologicalSpace](r: (T, T) -> Bool, c: Set[T] -> Bool) {
    (forall(s: Set[T]) { c(s) implies is_quotient_open[T](r, s) })
        implies is_quotient_open[T](r, big_union(c))
} by {
    if forall(s: Set[T]) { c(s) implies is_quotient_open[T](r, s) } {
        forall(s: Set[T]) {
            if c(s) {
                is_quotient_open[T](r, s)
                T.is_open(s)
            }
        }
        open_big_union[T](c)
        T.is_open(big_union(c))
        forall(x: T, y: T) {
            if big_union(c).contains(x) and r(x, y) {
                big_union_contains_eq(c, x)
                let s: Set[T] satisfy {
                    c(s) and s.contains(x)
                }
                is_quotient_open[T](r, s)
                is_saturated_under(s, r)
                is_saturated_under(s, r) = forall(x1: T, y1: T) {
                    s.contains(x1) and r(x1, y1) implies s.contains(y1)
                }
                s.contains(y)
                big_union_contains_of_member(c, s, y)
                big_union(c).contains(y)
            }
        }
        is_saturated_under(big_union(c), r) = forall(x: T, y: T) {
            big_union(c).contains(x) and r(x, y) implies big_union(c).contains(y)
        }
        is_saturated_under(big_union(c), r)
        T.is_open(big_union(c)) and is_saturated_under(big_union(c), r)
        is_quotient_open[T](r, big_union(c)) = (
            T.is_open(big_union(c)) and is_saturated_under(big_union(c), r)
        )
        is_quotient_open[T](r, big_union(c))
    }
}


/// A four-element inductive type used to demonstrate the discrete topology
/// on a non-trivial type.
inductive DiscreteFour {
    /// First point.
    d0

    /// Second point.
    d1

    /// Third point.
    d2

    /// Fourth point.
    d3
}

/// The discrete topology on DiscreteFour: every set is open.
define discrete_four_open(s: Set[DiscreteFour]) -> Bool {
    true
}

instance DiscreteFour: TopologicalSpace {
    let is_open: Set[DiscreteFour] -> Bool = discrete_four_open
}

/// On DiscreteFour every subset is open.
theorem discrete_four_every_set_open(s: Set[DiscreteFour]) {
    DiscreteFour.is_open(s)
}

/// In the discrete topology, every subset is closed (its complement is open).
theorem discrete_four_every_set_closed(s: Set[DiscreteFour]) {
    is_closed(s)
} by {
    discrete_four_every_set_open(s.c)
    DiscreteFour.is_open(s.c)
}

/// A two-element inductive type used to demonstrate the indiscrete topology
/// on a non-trivial type.
inductive IndiscreteTwo {
    /// First point.
    i0

    /// Second point.
    i1
}

/// The indiscrete topology on IndiscreteTwo: the only open sets are the empty
/// set and the whole space.
define indiscrete_two_open(s: Set[IndiscreteTwo]) -> Bool {
    s = Set[IndiscreteTwo].empty_set or s = Set[IndiscreteTwo].universal_set
}

/// The intersection of two indiscrete-open sets is indiscrete-open.
theorem indiscrete_two_inter(s: Set[IndiscreteTwo], t: Set[IndiscreteTwo]) {
    indiscrete_two_open(s) and indiscrete_two_open(t) implies indiscrete_two_open(s.intersection(t))
} by {
    if indiscrete_two_open(s) and indiscrete_two_open(t) {
        s = Set[IndiscreteTwo].empty_set or s = Set[IndiscreteTwo].universal_set
        t = Set[IndiscreteTwo].empty_set or t = Set[IndiscreteTwo].universal_set
        if s = Set[IndiscreteTwo].empty_set {
            intersection_with_empty_is_empty[IndiscreteTwo](t)
            s.intersection(t) = t.intersection(Set[IndiscreteTwo].empty_set)
            intersection_comm[IndiscreteTwo](s, t)
            s.intersection(t) = Set[IndiscreteTwo].empty_set
            indiscrete_two_open(s.intersection(t))
        } else {
            s = Set[IndiscreteTwo].universal_set
            if t = Set[IndiscreteTwo].empty_set {
                intersection_with_empty_is_empty[IndiscreteTwo](s)
                s.intersection(t) = Set[IndiscreteTwo].empty_set
                indiscrete_two_open(s.intersection(t))
            } else {
                t = Set[IndiscreteTwo].universal_set
                intersection_with_universal_is_self[IndiscreteTwo](s)
                s.intersection(t) = s
                s.intersection(t) = Set[IndiscreteTwo].universal_set
                indiscrete_two_open(s.intersection(t))
            }
        }
    }
}

/// The big union of a family of indiscrete-open sets is indiscrete-open.
theorem indiscrete_two_big_union(c: Set[IndiscreteTwo] -> Bool) {
    (forall(s: Set[IndiscreteTwo]) { c(s) implies indiscrete_two_open(s) })
        implies indiscrete_two_open(big_union(c))
} by {
    if forall(s: Set[IndiscreteTwo]) { c(s) implies indiscrete_two_open(s) } {
        if c(Set[IndiscreteTwo].universal_set) {
            big_union_with_universal_member[IndiscreteTwo](c)
            big_union(c) = Set[IndiscreteTwo].universal_set
            indiscrete_two_open(big_union(c))
        } else {
            forall(s: Set[IndiscreteTwo]) {
                if c(s) {
                    indiscrete_two_open(s)
                    s = Set[IndiscreteTwo].empty_set or s = Set[IndiscreteTwo].universal_set
                    s = Set[IndiscreteTwo].empty_set
                }
            }
            big_union_of_empty_family[IndiscreteTwo](c)
            big_union(c) = Set[IndiscreteTwo].empty_set
            indiscrete_two_open(big_union(c))
        }
    }
}

instance IndiscreteTwo: TopologicalSpace {
    let is_open: Set[IndiscreteTwo] -> Bool = indiscrete_two_open
}

/// In the indiscrete topology, the only open sets are empty and the whole space.
theorem indiscrete_two_open_iff(s: Set[IndiscreteTwo]) {
    IndiscreteTwo.is_open(s) = (s = Set[IndiscreteTwo].empty_set or s = Set[IndiscreteTwo].universal_set)
}
