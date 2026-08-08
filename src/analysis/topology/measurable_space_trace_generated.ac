from data.basic.set import Set, sets_subset_intersection
from analysis.topology.measurable_space import SigmaAlgebra, sigma_algebra_le,
    generated_sigma_algebra, generated_sigma_algebra_contains_generator,
    generated_sigma_algebra_is_smallest,
    is_measurable_in_subspace, subspace_measurable_from_ambient,
    trace_measurable, trace_sigma_algebra,
    trace_sigma_algebra_is_measurable_eq,
    trace_sigma_algebra_measurable_iff_subspace

/// Every ambient-measurable set is measurable in the trace sigma-algebra.
theorem trace_sigma_algebra_measurable_from_ambient[T](
        m: SigmaAlgebra[T], a: Set[T], v: Set[T]) {
    m.is_measurable(v) implies trace_sigma_algebra(m, a).is_measurable(v)
} by {
    if m.is_measurable(v) {
        subspace_measurable_from_ambient(m, a, v)
        trace_measurable(m, a, v)
        trace_sigma_algebra_is_measurable_eq(m, a, v)
        trace_sigma_algebra(m, a).is_measurable(v)
    }
}

/// The trace of an ambient-measurable set is measurable in the trace sigma-algebra.
theorem trace_sigma_algebra_measurable_intersection_from_ambient[T](
        m: SigmaAlgebra[T], a: Set[T], v: Set[T]) {
    m.is_measurable(v) implies
        trace_sigma_algebra(m, a).is_measurable(v.intersection(a))
} by {
    if m.is_measurable(v) {
        subspace_measurable_from_ambient(m, a, v)
        sets_subset_intersection(v, a)
        trace_sigma_algebra_measurable_iff_subspace(m, a, v.intersection(a))
        trace_sigma_algebra(m, a).is_measurable(v.intersection(a))
    }
}

/// The trace construction is monotone in the ambient sigma-algebra.
theorem trace_sigma_algebra_mono[T](m1: SigmaAlgebra[T], m2: SigmaAlgebra[T],
        a: Set[T]) {
    sigma_algebra_le(m1, m2) implies
        sigma_algebra_le(trace_sigma_algebra(m1, a), trace_sigma_algebra(m2, a))
} by {
    if sigma_algebra_le(m1, m2) {
        forall(u: Set[T]) {
            if trace_sigma_algebra(m1, a).is_measurable(u) {
                trace_sigma_algebra_is_measurable_eq(m1, a, u)
                let v: Set[T] satisfy {
                    m1.is_measurable(v) and u.intersection(a) = v.intersection(a)
                }
                sigma_algebra_le(m1, m2) = forall(s: Set[T]) {
                    m1.is_measurable(s) implies m2.is_measurable(s)
                }
                is_measurable_in_subspace(m2, a, u.intersection(a))
                trace_measurable(m2, a, u)
                trace_sigma_algebra_is_measurable_eq(m2, a, u)
                trace_sigma_algebra(m2, a).is_measurable(u)
            }
        }
    }
}

/// The generator obtained by intersecting generator sets with a fixed carrier.
define trace_generator[T](a: Set[T], g: Set[T] -> Bool) -> (Set[T] -> Bool) {
    function(u: Set[T]) {
        exists(s: Set[T]) { g(s) and u = s.intersection(a) }
    }
}

/// Every traced generator is measurable in the trace of the generated sigma-algebra.
theorem trace_generated_sigma_algebra_contains_trace_generator[T](
        g: Set[T] -> Bool, a: Set[T], u: Set[T]) {
    trace_generator(a, g)(u) implies
        trace_sigma_algebra(generated_sigma_algebra(g), a).is_measurable(u)
} by {
    if trace_generator(a, g)(u) {
        let s: Set[T] satisfy {
            g(s) and u = s.intersection(a)
        }
        generated_sigma_algebra_contains_generator(g, s)
        trace_sigma_algebra_measurable_intersection_from_ambient(
            generated_sigma_algebra(g), a, s)
        trace_sigma_algebra(generated_sigma_algebra(g), a).is_measurable(
            s.intersection(a))
        trace_sigma_algebra(generated_sigma_algebra(g), a).is_measurable(u)
    }
}

/// The trace sigma-algebra contains all traced generators.
theorem trace_generated_sigma_algebra_contains_all_trace_generators[T](
        g: Set[T] -> Bool, a: Set[T]) {
    forall(u: Set[T]) {
        trace_generator(a, g)(u) implies
            trace_sigma_algebra(generated_sigma_algebra(g), a).is_measurable(u)
    }
} by {
    forall(u: Set[T]) {
        if trace_generator(a, g)(u) {
            trace_generated_sigma_algebra_contains_trace_generator(g, a, u)
            trace_sigma_algebra(generated_sigma_algebra(g), a).is_measurable(u)
        }
    }
}

/// The sigma-algebra generated by traced generators is below the trace of the
/// sigma-algebra generated by the original generators.
theorem generated_trace_sigma_algebra_le_trace_generated_sigma_algebra[T](
        g: Set[T] -> Bool, a: Set[T]) {
    sigma_algebra_le(generated_sigma_algebra(trace_generator(a, g)),
        trace_sigma_algebra(generated_sigma_algebra(g), a))
} by {
    trace_generated_sigma_algebra_contains_all_trace_generators(g, a)
    generated_sigma_algebra_is_smallest(trace_generator(a, g),
        trace_sigma_algebra(generated_sigma_algebra(g), a))
}

/// A generator set gives a traced generator by intersection with the carrier.
theorem trace_generator_intro[T](g: Set[T] -> Bool, a: Set[T], s: Set[T]) {
    g(s) implies trace_generator(a, g)(s.intersection(a))
} by {
    if g(s) {
        g(s) and s.intersection(a) = s.intersection(a)
        exists(t: Set[T]) {
            g(t) and s.intersection(a) = t.intersection(a)
        }
        trace_generator(a, g)(s.intersection(a))
    }
}

/// The generated trace sigma-algebra contains each traced generator set.
theorem generated_trace_sigma_algebra_contains_generator_trace[T](
        g: Set[T] -> Bool, a: Set[T], s: Set[T]) {
    g(s) implies generated_sigma_algebra(trace_generator(a, g)).is_measurable(s.intersection(a))
} by {
    if g(s) {
        trace_generator_intro(g, a, s)
        trace_generator(a, g)(s.intersection(a))
        generated_sigma_algebra_contains_generator(trace_generator(a, g), s.intersection(a))
    }
}

/// The trace of the generated sigma-algebra contains each traced generator set.
theorem trace_generated_sigma_algebra_contains_generator_trace[T](
        g: Set[T] -> Bool, a: Set[T], s: Set[T]) {
    g(s) implies trace_sigma_algebra(generated_sigma_algebra(g), a).is_measurable(s.intersection(a))
} by {
    if g(s) {
        trace_generator_intro(g, a, s)
        trace_generated_sigma_algebra_contains_trace_generator(g, a, s.intersection(a))
    }
}
