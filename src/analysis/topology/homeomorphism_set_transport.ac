from data.basic.set import Set, set_image, set_preimage, maps_into_set_image,
    set_image_contains_witness, set_image_preimage_subset, subset_set_preimage_image,
    set_image_monotone, set_preimage_monotone, set_image_subset_imp_subset_of_injective,
    set_preimage_subset_imp_subset_of_surjective, set_preimage_contains_eq,
    set_preimage_compl, subset_contains_eq, set_ext, double_inclusion,
    set_eq_transport_predicate_rev
from data.basic.functions import is_injective_fn, is_surjective_fn, is_bijection_fn,
    is_left_inverse_fn, is_right_inverse_fn, left_inverse_fn_imp_injective_fn,
    right_inverse_fn_imp_surjective_fn
from analysis.topology.topological_space import TopologicalSpace, is_closed, is_continuous,
    is_homeomorphism, is_homeomorphism_left_inv, is_homeomorphism_right_inv,
    is_homeomorphism_continuous_fwd, is_homeomorphism_continuous_inv
from analysis.topology.continuous_preimage import continuous_open_preimage

/// The forward map of a homeomorphism is injective.
theorem homeomorphism_forward_injective[X: TopologicalSpace, Y: TopologicalSpace](
    f: X -> Y, g: Y -> X) {
    is_homeomorphism[X, Y](f, g) implies is_injective_fn(f)
} by {
    if is_homeomorphism[X, Y](f, g) {
        forall(x: X) {
            is_homeomorphism_left_inv[X, Y](f, g, x)
            g(f(x)) = x
        }
        left_inverse_fn_imp_injective_fn(f, g)
        is_injective_fn(f)
    }
}

/// The forward map of a homeomorphism is surjective.
theorem homeomorphism_forward_surjective[X: TopologicalSpace, Y: TopologicalSpace](
    f: X -> Y, g: Y -> X) {
    is_homeomorphism[X, Y](f, g) implies is_surjective_fn(f)
} by {
    if is_homeomorphism[X, Y](f, g) {
        forall(y: Y) {
            is_homeomorphism_right_inv[X, Y](f, g, y)
            f(g(y)) = y
        }
        right_inverse_fn_imp_surjective_fn(f, g)
        is_surjective_fn(f)
    }
}

/// The forward map of a homeomorphism is bijective.
theorem homeomorphism_forward_bijection[X: TopologicalSpace, Y: TopologicalSpace](
    f: X -> Y, g: Y -> X) {
    is_homeomorphism[X, Y](f, g) implies is_bijection_fn(f)
} by {
    if is_homeomorphism[X, Y](f, g) {
        homeomorphism_forward_injective[X, Y](f, g)
        homeomorphism_forward_surjective[X, Y](f, g)
        is_bijection_fn(f)
    }
}

/// The image of a source set under a homeomorphism is the preimage of that set under the inverse map.
theorem homeomorphism_image_eq_inverse_preimage[X: TopologicalSpace, Y: TopologicalSpace](
    f: X -> Y, g: Y -> X, s: Set[X]) {
    is_homeomorphism[X, Y](f, g) implies set_image(s, f) = set_preimage(g, s)
} by {
    let image = set_image(s, f)
    let pre = set_preimage(g, s)
    if is_homeomorphism[X, Y](f, g) {
        forall(y: Y) {
            if image.contains(y) {
                set_image_contains_witness(s, f, y)
                let x: X satisfy {
                    s.contains(x) and y = f(x)
                }
                is_homeomorphism_left_inv[X, Y](f, g, x)
                set_preimage_contains_eq(g, s, y)
                pre.contains(y)
            }
            if pre.contains(y) {
                set_preimage_contains_eq(g, s, y)
                s.contains(g(y))
                is_homeomorphism_right_inv[X, Y](f, g, y)
                f(g(y)) = y
                maps_into_set_image(s, f, g(y))
                image.contains(y)
            }
            image.contains(y) = pre.contains(y)
        }
        set_ext(image, pre)
        set_image(s, f) = set_preimage(g, s)
    }
}

/// The preimage of a target set under a homeomorphism is the image of that set under the inverse map.
theorem homeomorphism_preimage_eq_inverse_image[X: TopologicalSpace, Y: TopologicalSpace](
    f: X -> Y, g: Y -> X, t: Set[Y]) {
    is_homeomorphism[X, Y](f, g) implies set_preimage(f, t) = set_image(t, g)
} by {
    let pre = set_preimage(f, t)
    let image = set_image(t, g)
    if is_homeomorphism[X, Y](f, g) {
        forall(x: X) {
            if pre.contains(x) {
                set_preimage_contains_eq(f, t, x)
                t.contains(f(x))
                is_homeomorphism_left_inv[X, Y](f, g, x)
                g(f(x)) = x
                maps_into_set_image(t, g, f(x))
                image.contains(x)
            }
            if image.contains(x) {
                set_image_contains_witness(t, g, x)
                let y: Y satisfy {
                    t.contains(y) and x = g(y)
                }
                is_homeomorphism_right_inv[X, Y](f, g, y)
                set_preimage_contains_eq(f, t, x)
                pre.contains(x)
            }
            pre.contains(x) = image.contains(x)
        }
        set_ext(pre, image)
        set_preimage(f, t) = set_image(t, g)
    }
}

/// A homeomorphism maps the preimage of a target set exactly onto that target set.
theorem homeomorphism_image_preimage_eq[X: TopologicalSpace, Y: TopologicalSpace](
    f: X -> Y, g: Y -> X, t: Set[Y]) {
    is_homeomorphism[X, Y](f, g) implies set_image(set_preimage(f, t), f) = t
} by {
    let image = set_image(set_preimage(f, t), f)
    if is_homeomorphism[X, Y](f, g) {
        set_image_preimage_subset(t, f)
        subset_contains_eq(t, image)
        forall(y: Y) {
            if t.contains(y) {
                is_homeomorphism_right_inv[X, Y](f, g, y)
                f(g(y)) = y
                set_preimage_contains_eq(f, t, g(y))
                set_preimage(f, t).contains(g(y))
                maps_into_set_image(set_preimage(f, t), f, g(y))
                image.contains(y)
            }
        }
        t.subset(image)
        double_inclusion(image, t)
    }
}

/// The preimage under a homeomorphism of the image of a source set is exactly that source set.
theorem homeomorphism_preimage_image_eq[X: TopologicalSpace, Y: TopologicalSpace](
    f: X -> Y, g: Y -> X, s: Set[X]) {
    is_homeomorphism[X, Y](f, g) implies set_preimage(f, set_image(s, f)) = s
} by {
    let pre = set_preimage(f, set_image(s, f))
    if is_homeomorphism[X, Y](f, g) {
        subset_set_preimage_image(s, f)
        subset_contains_eq(pre, s)
        forall(x: X) {
            if pre.contains(x) {
                set_preimage_contains_eq(f, set_image(s, f), x)
                set_image_contains_witness(s, f, f(x))
                let z: X satisfy {
                    s.contains(z) and f(x) = f(z)
                }
                is_homeomorphism_left_inv[X, Y](f, g, x)
                is_homeomorphism_left_inv[X, Y](f, g, z)
                g(f(x)) = x
                g(f(z)) = z
                s.contains(x)
            }
        }
        pre.subset(s)
        double_inclusion(pre, s)
    }
}

/// A homeomorphism transports source-set inclusion to inclusion of forward images, and reflects it back.
theorem homeomorphism_image_subset_iff[X: TopologicalSpace, Y: TopologicalSpace](
    f: X -> Y, g: Y -> X, a: Set[X], b: Set[X]) {
    is_homeomorphism[X, Y](f, g) implies
    (set_image(a, f).subset(set_image(b, f)) = a.subset(b))
} by {
    if is_homeomorphism[X, Y](f, g) {
        if set_image(a, f).subset(set_image(b, f)) {
            homeomorphism_forward_injective[X, Y](f, g)
            set_image_subset_imp_subset_of_injective(a, b, f)
            a.subset(b)
        }
        if a.subset(b) {
            set_image_monotone(a, b, f)
            set_image(a, f).subset(set_image(b, f))
        }
        set_image(a, f).subset(set_image(b, f)) = a.subset(b)
    }
}

/// A homeomorphism transports target-set inclusion to inclusion of forward preimages, and reflects it back.
theorem homeomorphism_preimage_subset_iff[X: TopologicalSpace, Y: TopologicalSpace](
    f: X -> Y, g: Y -> X, a: Set[Y], b: Set[Y]) {
    is_homeomorphism[X, Y](f, g) implies
    (set_preimage(f, a).subset(set_preimage(f, b)) = a.subset(b))
} by {
    if is_homeomorphism[X, Y](f, g) {
        if set_preimage(f, a).subset(set_preimage(f, b)) {
            homeomorphism_forward_surjective[X, Y](f, g)
            set_preimage_subset_imp_subset_of_surjective(a, b, f)
            a.subset(b)
        }
        if a.subset(b) {
            set_preimage_monotone(f, a, b)
            set_preimage(f, a).subset(set_preimage(f, b))
        }
        set_preimage(f, a).subset(set_preimage(f, b)) = a.subset(b)
    }
}

/// The preimage of an open set under the forward map of a homeomorphism is open.
theorem homeomorphism_preimage_open[X: TopologicalSpace, Y: TopologicalSpace](
    f: X -> Y, g: Y -> X, t: Set[Y]) {
    is_homeomorphism[X, Y](f, g) and Y.is_open(t) implies X.is_open(set_preimage(f, t))
} by {
    if is_homeomorphism[X, Y](f, g) and Y.is_open(t) {
        is_homeomorphism_continuous_fwd[X, Y](f, g)
        continuous_open_preimage[X, Y](f, t)
        X.is_open(set_preimage(f, t))
    }
}

/// The preimage of a closed set under the forward map of a homeomorphism is closed.
theorem homeomorphism_preimage_closed[X: TopologicalSpace, Y: TopologicalSpace](
    f: X -> Y, g: Y -> X, t: Set[Y]) {
    is_homeomorphism[X, Y](f, g) and is_closed[Y](t) implies is_closed[X](set_preimage(f, t))
} by {
    if is_homeomorphism[X, Y](f, g) and is_closed[Y](t) {
        homeomorphism_preimage_open[X, Y](f, g, t.c)
        X.is_open(set_preimage(f, t.c))
        set_preimage_compl(f, t)
        is_closed[X](set_preimage(f, t))
    }
}

/// A homeomorphism sends open source sets to open target sets.
theorem homeomorphism_image_open[X: TopologicalSpace, Y: TopologicalSpace](
    f: X -> Y, g: Y -> X, s: Set[X]) {
    is_homeomorphism[X, Y](f, g) and X.is_open(s) implies Y.is_open(set_image(s, f))
} by {
    if is_homeomorphism[X, Y](f, g) and X.is_open(s) {
        is_homeomorphism_continuous_inv[X, Y](f, g)
        continuous_open_preimage[Y, X](g, s)
        Y.is_open(set_preimage(g, s))
        homeomorphism_image_eq_inverse_preimage[X, Y](f, g, s)
        let image = set_image(s, f)
        let pre = set_preimage(g, s)
        image = pre
        let p: Set[Y] -> Bool = function(u: Set[Y]) { Y.is_open(u) }
        set_eq_transport_predicate_rev[Y](p, image, pre)
        Y.is_open(set_image(s, f))
    }
}

/// A homeomorphism sends closed source sets to closed target sets.
theorem homeomorphism_image_closed[X: TopologicalSpace, Y: TopologicalSpace](
    f: X -> Y, g: Y -> X, s: Set[X]) {
    is_homeomorphism[X, Y](f, g) and is_closed[X](s) implies is_closed[Y](set_image(s, f))
} by {
    if is_homeomorphism[X, Y](f, g) and is_closed[X](s) {
        is_homeomorphism_continuous_inv[X, Y](f, g)
        continuous_open_preimage[Y, X](g, s.c)
        Y.is_open(set_preimage(g, s.c))
        set_preimage_compl(g, s)
        homeomorphism_image_eq_inverse_preimage[X, Y](f, g, s)
        let image = set_image(s, f)
        let pre = set_preimage(g, s)
        image = pre
        let p: Set[Y] -> Bool = function(u: Set[Y]) { is_closed[Y](u) }
        set_eq_transport_predicate_rev[Y](p, image, pre)
        is_closed[Y](set_image(s, f))
    }
}
