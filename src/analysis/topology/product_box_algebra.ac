from data.basic.set import Set, set_ext, intersection_contains_eq, empty_set_contains_eq
from pair import Pair
from analysis.topology.topological_space import TopologicalSpace, box, box_contains_eq

/// The intersection of two boxes is the box of the intersections of the factors.
theorem box_inter[X, Y](
    u1: Set[X], v1: Set[Y], u2: Set[X], v2: Set[Y]) {
    box(u1, v1).intersection(box(u2, v2)) = box(u1.intersection(u2), v1.intersection(v2))
} by {
    let l = box(u1, v1)
    let r = box(u2, v2)
    forall(p: Pair[X, Y]) {
        box_contains_eq(u1, v1, p)
        box_contains_eq(u2, v2, p)
        box_contains_eq(u1.intersection(u2), v1.intersection(v2), p)
        intersection_contains_eq(l, r, p)
        intersection_contains_eq(u1, u2, p.first)
        intersection_contains_eq(v1, v2, p.second)
        l.intersection(r).contains(p) = box(u1.intersection(u2), v1.intersection(v2)).contains(p)
    }
    set_ext(l.intersection(r), box(u1.intersection(u2), v1.intersection(v2)))
}

/// A box with an empty first factor is empty.
theorem box_empty_left[X, Y](v: Set[Y]) {
    box(Set[X].empty_set, v) = Set[Pair[X, Y]].empty_set
} by {
    forall(p: Pair[X, Y]) {
        box_contains_eq(Set[X].empty_set, v, p)
        empty_set_contains_eq[X](p.first)
        empty_set_contains_eq[Pair[X, Y]](p)
        box(Set[X].empty_set, v).contains(p) = Set[Pair[X, Y]].empty_set.contains(p)
    }
    set_ext(box(Set[X].empty_set, v), Set[Pair[X, Y]].empty_set)
}

/// A box with an empty second factor is empty.
theorem box_empty_right[X, Y](u: Set[X]) {
    box(u, Set[Y].empty_set) = Set[Pair[X, Y]].empty_set
} by {
    forall(p: Pair[X, Y]) {
        box_contains_eq(u, Set[Y].empty_set, p)
        empty_set_contains_eq[Y](p.second)
        empty_set_contains_eq[Pair[X, Y]](p)
        box(u, Set[Y].empty_set).contains(p) = Set[Pair[X, Y]].empty_set.contains(p)
    }
    set_ext(box(u, Set[Y].empty_set), Set[Pair[X, Y]].empty_set)
}
