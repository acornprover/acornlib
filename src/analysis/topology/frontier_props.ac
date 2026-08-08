from data.basic.set import Set, set_ext, intersection_contains_eq,
    empty_set_contains_eq, sets_subset_intersection, difference_subset,
    difference_contains_imp_contains, difference_contains_imp_not_contains,
    intersection_with_superset_is_self, intersection_comm,
    intersection_union_difference_is_self
from analysis.topology.topological_space import TopologicalSpace, closure, interior, frontier,
    frontier_eq_closure_inter_closure_compl, interior_subset_closure

/// Generic set identity: a set absorbs the part of a superset outside it.
theorem subset_union_difference_eq[K](a: Set[K], b: Set[K]) {
    a.subset(b) implies a.union(b.difference(a)) = b
} by {
    if a.subset(b) {
        intersection_with_superset_is_self[K](a, b)
        intersection_comm[K](a, b)
        intersection_union_difference_is_self[K](b, a)
        b.intersection(a).union(b.difference(a)) = b
        a.union(b.difference(a)) = b
    }
}

/// Generic set identity: a set is disjoint from the part of any set outside it.
theorem inter_difference_self_empty[K](a: Set[K], b: Set[K]) {
    a.intersection(b.difference(a)) = Set[K].empty_set
} by {
    let l = a.intersection(b.difference(a))
    forall(x: K) {
        intersection_contains_eq(a, b.difference(a), x)
        empty_set_contains_eq[K](x)
        if l.contains(x) {
            a.contains(x)
            b.difference(a).contains(x)
            difference_contains_imp_not_contains[K](b, a, x)
            not a.contains(x)
            false
        }
        not l.contains(x)
        l.contains(x) = Set[K].empty_set.contains(x)
    }
    set_ext(l, Set[K].empty_set)
}

/// The frontier of a set is contained in its closure.
theorem frontier_subset_closure[T: TopologicalSpace](s: Set[T]) {
    frontier(s).subset(closure(s))
} by {
    frontier(s) = closure(s).difference(interior(s))
    difference_subset[T](closure(s), interior(s))
    closure(s).difference(interior(s)).subset(closure(s))
}

/// The frontier of a set is contained in the closure of its complement.
theorem frontier_subset_closure_compl[T: TopologicalSpace](s: Set[T]) {
    frontier(s).subset(closure(s.c))
} by {
    frontier_eq_closure_inter_closure_compl[T](s)
    frontier(s) = closure(s).intersection(closure(s.c))
    sets_subset_intersection(closure(s), closure(s.c))
    closure(s).intersection(closure(s.c)).subset(closure(s.c))
}

/// The closure of a set is the union of its interior and its frontier.
theorem closure_eq_interior_union_frontier[T: TopologicalSpace](s: Set[T]) {
    closure(s) = interior(s).union(frontier(s))
} by {
    frontier(s) = closure(s).difference(interior(s))
    interior_subset_closure[T](s)
    interior(s).subset(closure(s))
    subset_union_difference_eq[T](interior(s), closure(s))
    interior(s).union(closure(s).difference(interior(s))) = closure(s)
    interior(s).union(frontier(s)) = closure(s)
}

/// The interior and the frontier of a set are disjoint.
theorem interior_inter_frontier_empty[T: TopologicalSpace](s: Set[T]) {
    interior(s).intersection(frontier(s)) = Set[T].empty_set
} by {
    frontier(s) = closure(s).difference(interior(s))
    inter_difference_self_empty[T](interior(s), closure(s))
    interior(s).intersection(closure(s).difference(interior(s))) = Set[T].empty_set
}
