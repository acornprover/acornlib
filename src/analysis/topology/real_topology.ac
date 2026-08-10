/// Real-line topology.
///
/// This file develops the topology of the real line within the general
/// topological framework: `Real` carries the standard topology as a
/// `TopologicalSpace` instance (see `analysis.topology.borel_functions`), in
/// which a set is open exactly when every point of the set is an interior
/// point.  The order-theoretic intervals of `order_set` are connected to that
/// topology: open intervals are open sets, closed intervals are closed sets,
/// unions of open sets are open and intersections of closed sets are closed,
/// complements of open sets are closed and vice versa, closed intervals are
/// compact (Heine-Borel), and the real line is connected.

from data.basic.set import Set, set_ext, compl_contains_eq, intersection_contains_eq,
    intersection_contains_intro, union_contains_eq, union_comm, subset_contains,
    subset_trans, empty_set_contains_eq, universal_set_contains_eq,
    compl_of_compl_is_self
from list import List
from order import lt_of_lt_of_lte, lt_imp_lte, not_lt_self, lt_or_lte, min_lte_left,
    min_lte_right, lt_min_iff, open_interval, closed_interval
from order_set import open_interval_set, open_interval_set_contains_eq,
    closed_interval_set, closed_interval_set_contains_eq,
    closed_interval_set_lower_le, closed_interval_set_le_upper
from real import Real, lte_self, lte_trans, lt_trans, lt_lte_trans, lte_lt_trans,
    lte_or_gte, lte_both_ways_imp_eq, not_lt_imp_gte, gte_imp_not_lt, gt_imp_not_lte,
    not_lte_imp_gt,
    close_imp_bounds, bounds_imp_close,
    lt_imp_minus_pos, neg_pos_is_neg, neg_lt_zero,
    eps_lt_half, lt_add_pos, lt_add_right, sub_cancels, is_nonempty, is_set_upper_bound,
    has_upper_bound, is_set_supremum, completeness
from analysis.topology.topological_space import TopologicalSpace, is_closed,
    open_union_open, closed_inter, open_inter, open_big_union,
    big_union, big_union_contains_eq,
    compl_intersection_eq_union_compl,
    is_compact, is_compact_iff, empty_set_is_compact, is_connected, is_connected_iff,
    is_disconnection, is_disconnection_iff, is_open_cover,
    is_open_cover_iff, is_subfamily, is_subfamily_nil,
    is_subfamily_cons_mpr, list_union_of_sets, list_union_of_sets_nil,
    list_union_of_sets_cons
from analysis.topology.borel_functions import real_is_open, real_is_interior_point,
    real_is_interior_point_intro, real_open_is_interior_point

/// True if a real number lies strictly above a lower endpoint.
define above_point(lower: Real, x: Real) -> Bool {
    lower < x
}

/// True if a real number lies strictly below an upper endpoint.
define below_point(upper: Real, x: Real) -> Bool {
    x < upper
}

/// True if a real number lies at or above a lower endpoint.
define at_or_above_point(lower: Real, x: Real) -> Bool {
    lower <= x
}

/// True if a real number lies at or below an upper endpoint.
define at_or_below_point(upper: Real, x: Real) -> Bool {
    x <= upper
}

/// The open ray `(lower, +infinity)` as a set of real numbers.
define open_upper_ray_set(lower: Real) -> Set[Real] {
    Set[Real].new(above_point(lower))
}

/// The open ray `(-infinity, upper)` as a set of real numbers.
define open_lower_ray_set(upper: Real) -> Set[Real] {
    Set[Real].new(below_point(upper))
}

/// The closed ray `[lower, +infinity)` as a set of real numbers.
define closed_upper_ray_set(lower: Real) -> Set[Real] {
    Set[Real].new(at_or_above_point(lower))
}

/// The closed ray `(-infinity, upper]` as a set of real numbers.
define closed_lower_ray_set(upper: Real) -> Set[Real] {
    Set[Real].new(at_or_below_point(upper))
}

/// Membership in an open upper ray is strict inequality above the endpoint.
theorem open_upper_ray_set_contains_eq(lower: Real, x: Real) {
    open_upper_ray_set(lower).contains(x) = (lower < x)
}

/// Membership in an open lower ray is strict inequality below the endpoint.
theorem open_lower_ray_set_contains_eq(upper: Real, x: Real) {
    open_lower_ray_set(upper).contains(x) = (x < upper)
}

/// Membership in a closed upper ray is inequality above the endpoint.
theorem closed_upper_ray_set_contains_eq(lower: Real, x: Real) {
    closed_upper_ray_set(lower).contains(x) = (lower <= x)
}

/// Membership in a closed lower ray is inequality below the endpoint.
theorem closed_lower_ray_set_contains_eq(upper: Real, x: Real) {
    closed_lower_ray_set(upper).contains(x) = (x <= upper)
}

/// A positive summand below half of a bound is below that bound.
theorem positive_half_lt_bound(eps: Real, bound: Real) {
    eps.is_positive and eps + eps < bound implies eps < bound
} by {
    if eps.is_positive and eps + eps < bound {
        Real.0 < eps
        eps < eps + eps
        eps + eps < bound
        lt_trans(eps, eps + eps, bound)
        eps < bound
    }
}

/// Subtracting a positive amount gives a strictly smaller number.
theorem sub_lt_self_of_pos(a: Real, b: Real) {
    b.is_positive implies a - b < a
} by {
    if b.is_positive {
        neg_pos_is_neg(b)
        (-b).is_negative
        neg_lt_zero(-b)
        -b < Real.0
        lt_add_right(-b, Real.0, a)
        -b + a < Real.0 + a
        a - b < a
    }
}

/// Open upper rays are open in the standard topology of the real line.
theorem real_open_upper_ray_set(lower: Real) {
    real_is_open(open_upper_ray_set(lower))
} by {
    forall(x: Real) {
        if open_upper_ray_set(lower).contains(x) {
            open_upper_ray_set_contains_eq(lower, x)
            lower < x
            lt_imp_minus_pos(lower, x)
            let gap = x - lower
            gap.is_positive
            eps_lt_half(gap)
            let eps: Real satisfy {
                eps.is_positive and eps + eps < gap
            }
            eps.is_positive
            positive_half_lt_bound(eps, gap)
            eps < gap
            eps < x - lower
            lt_add_right(eps, x - lower, lower)
            eps + lower < x - lower + lower
            x - lower + lower = x
            eps + lower < x
            lower + eps < x
            lt_add_right(lower + eps, x, -eps)
            (lower + eps) + -eps < x + -eps
            sub_cancels(lower, eps)
            lower + eps - eps = lower
            lower < x - eps
            forall(y: Real) {
                if y.is_close(x, eps) {
                    close_imp_bounds(y, x, eps)
                    x - eps < y
                    lt_trans(lower, x - eps, y)
                    lower < y
                    open_upper_ray_set_contains_eq(lower, y)
                    open_upper_ray_set(lower).contains(y)
                }
            }
            real_is_interior_point_intro(open_upper_ray_set(lower), x, eps)
            real_is_interior_point(open_upper_ray_set(lower), x)
        }
    }
    real_is_open(open_upper_ray_set(lower))
}

/// Open lower rays are open in the standard topology of the real line.
theorem real_open_lower_ray_set(upper: Real) {
    real_is_open(open_lower_ray_set(upper))
} by {
    forall(x: Real) {
        if open_lower_ray_set(upper).contains(x) {
            open_lower_ray_set_contains_eq(upper, x)
            x < upper
            lt_imp_minus_pos(x, upper)
            let gap = upper - x
            gap.is_positive
            eps_lt_half(gap)
            let eps: Real satisfy {
                eps.is_positive and eps + eps < gap
            }
            eps.is_positive
            positive_half_lt_bound(eps, gap)
            eps < gap
            eps < upper - x
            lt_add_right(eps, upper - x, x)
            eps + x < upper - x + x
            upper - x + x = upper
            eps + x < upper
            x + eps < upper
            forall(y: Real) {
                if y.is_close(x, eps) {
                    close_imp_bounds(y, x, eps)
                    y < x + eps
                    lt_trans(y, x + eps, upper)
                    y < upper
                    open_lower_ray_set_contains_eq(upper, y)
                    open_lower_ray_set(upper).contains(y)
                }
            }
            real_is_interior_point_intro(open_lower_ray_set(upper), x, eps)
            real_is_interior_point(open_lower_ray_set(upper), x)
        }
    }
    real_is_open(open_lower_ray_set(upper))
}

/// A closed upper ray is the complement of the corresponding open lower ray.
theorem closed_upper_ray_set_eq_complement_open_lower_ray_set(lower: Real) {
    closed_upper_ray_set(lower) = open_lower_ray_set(lower).c
} by {
    forall(x: Real) {
        if closed_upper_ray_set(lower).contains(x) {
            closed_upper_ray_set_contains_eq(lower, x)
            lower <= x
            not x < lower
            open_lower_ray_set_contains_eq(lower, x)
            not open_lower_ray_set(lower).contains(x)
            compl_contains_eq(open_lower_ray_set(lower), x)
            open_lower_ray_set(lower).c.contains(x)
        }
        if open_lower_ray_set(lower).c.contains(x) {
            compl_contains_eq(open_lower_ray_set(lower), x)
            not open_lower_ray_set(lower).contains(x)
            open_lower_ray_set_contains_eq(lower, x)
            not x < lower
            lower <= x
            closed_upper_ray_set_contains_eq(lower, x)
            closed_upper_ray_set(lower).contains(x)
        }
        closed_upper_ray_set(lower).contains(x) = open_lower_ray_set(lower).c.contains(x)
    }
    set_ext(closed_upper_ray_set(lower), open_lower_ray_set(lower).c)
}

/// A closed lower ray is the complement of the corresponding open upper ray.
theorem closed_lower_ray_set_eq_complement_open_upper_ray_set(upper: Real) {
    closed_lower_ray_set(upper) = open_upper_ray_set(upper).c
} by {
    forall(x: Real) {
        if closed_lower_ray_set(upper).contains(x) {
            closed_lower_ray_set_contains_eq(upper, x)
            x <= upper
            not upper < x
            open_upper_ray_set_contains_eq(upper, x)
            not open_upper_ray_set(upper).contains(x)
            compl_contains_eq(open_upper_ray_set(upper), x)
            open_upper_ray_set(upper).c.contains(x)
        }
        if open_upper_ray_set(upper).c.contains(x) {
            compl_contains_eq(open_upper_ray_set(upper), x)
            not open_upper_ray_set(upper).contains(x)
            open_upper_ray_set_contains_eq(upper, x)
            not upper < x
            x <= upper
            closed_lower_ray_set_contains_eq(upper, x)
            closed_lower_ray_set(upper).contains(x)
        }
        closed_lower_ray_set(upper).contains(x) = open_upper_ray_set(upper).c.contains(x)
    }
    set_ext(closed_lower_ray_set(upper), open_upper_ray_set(upper).c)
}

/// An open interval is the intersection of its two open bounding rays.
theorem open_interval_set_eq_ray_intersection(lower: Real, upper: Real) {
    open_interval_set(lower, upper) = open_upper_ray_set(lower).intersection(open_lower_ray_set(upper))
} by {
    let interval = open_interval_set(lower, upper)
    let rays = open_upper_ray_set(lower).intersection(open_lower_ray_set(upper))
    forall(x: Real) {
        if interval.contains(x) {
            open_interval_set_contains_eq(lower, upper, x)
            open_interval(lower, upper, x)
            lower < x
            x < upper
            open_upper_ray_set_contains_eq(lower, x)
            open_lower_ray_set_contains_eq(upper, x)
            open_upper_ray_set(lower).contains(x)
            open_lower_ray_set(upper).contains(x)
            intersection_contains_intro(open_upper_ray_set(lower), open_lower_ray_set(upper), x)
            rays.contains(x)
        }
        if rays.contains(x) {
            intersection_contains_eq(open_upper_ray_set(lower), open_lower_ray_set(upper), x)
            open_upper_ray_set(lower).contains(x)
            open_lower_ray_set(upper).contains(x)
            open_upper_ray_set_contains_eq(lower, x)
            open_lower_ray_set_contains_eq(upper, x)
            lower < x
            x < upper
            open_interval(lower, upper, x)
            open_interval_set_contains_eq(lower, upper, x)
            interval.contains(x)
        }
        interval.contains(x) = rays.contains(x)
    }
    set_ext(interval, rays)
}

/// A closed interval is the intersection of its two closed bounding rays.
theorem closed_interval_set_eq_ray_intersection(lower: Real, upper: Real) {
    closed_interval_set(lower, upper) = closed_upper_ray_set(lower).intersection(closed_lower_ray_set(upper))
} by {
    let interval = closed_interval_set(lower, upper)
    let rays = closed_upper_ray_set(lower).intersection(closed_lower_ray_set(upper))
    forall(x: Real) {
        if interval.contains(x) {
            closed_interval_set_contains_eq(lower, upper, x)
            closed_interval(lower, upper, x)
            lower <= x
            x <= upper
            closed_upper_ray_set_contains_eq(lower, x)
            closed_lower_ray_set_contains_eq(upper, x)
            closed_upper_ray_set(lower).contains(x)
            closed_lower_ray_set(upper).contains(x)
            intersection_contains_intro(closed_upper_ray_set(lower), closed_lower_ray_set(upper), x)
            rays.contains(x)
        }
        if rays.contains(x) {
            intersection_contains_eq(closed_upper_ray_set(lower), closed_lower_ray_set(upper), x)
            closed_upper_ray_set(lower).contains(x)
            closed_lower_ray_set(upper).contains(x)
            closed_upper_ray_set_contains_eq(lower, x)
            closed_lower_ray_set_contains_eq(upper, x)
            lower <= x
            x <= upper
            closed_interval(lower, upper, x)
            closed_interval_set_contains_eq(lower, upper, x)
            interval.contains(x)
        }
        interval.contains(x) = rays.contains(x)
    }
    set_ext(interval, rays)
}

/// The complement of an open set is closed.
theorem real_complement_of_open_is_closed(s: Set[Real]) {
    Real.is_open(s) implies is_closed[Real](s.c)
} by {
    if Real.is_open(s) {
        compl_of_compl_is_self(s)
        s.c.c = s
        is_closed[Real](s.c) = Real.is_open(s.c.c)
        Real.is_open(s.c.c)
        is_closed[Real](s.c)
    }
}

/// The complement of a closed set is open.
theorem real_complement_of_closed_is_open(s: Set[Real]) {
    is_closed[Real](s) implies Real.is_open(s.c)
} by {
    if is_closed[Real](s) {
        is_closed[Real](s) = Real.is_open(s.c)
        Real.is_open(s.c)
    }
}

/// The union of two open sets is open.
theorem real_union_of_open_is_open(s: Set[Real], t: Set[Real]) {
    Real.is_open(s) and Real.is_open(t) implies Real.is_open(s.union(t))
} by {
    if Real.is_open(s) and Real.is_open(t) {
        open_union_open[Real](s, t)
        Real.is_open(s.union(t))
    }
}

/// The intersection of two closed sets is closed.
theorem real_intersection_of_closed_is_closed(s: Set[Real], t: Set[Real]) {
    is_closed[Real](s) and is_closed[Real](t) implies is_closed[Real](s.intersection(t))
} by {
    if is_closed[Real](s) and is_closed[Real](t) {
        closed_inter[Real](s, t)
        is_closed[Real](s.intersection(t))
    }
}

/// The union of any family of open sets is open.
theorem real_big_union_of_open_is_open(c: Set[Real] -> Bool) {
    (forall(s: Set[Real]) { c(s) implies Real.is_open(s) })
        implies Real.is_open(big_union(c))
} by {
    if forall(s: Set[Real]) { c(s) implies Real.is_open(s) } {
        open_big_union[Real](c)
        Real.is_open(big_union(c))
    }
}

/// Closed upper rays are closed sets.
theorem real_closed_upper_ray_set(lower: Real) {
    is_closed[Real](closed_upper_ray_set(lower))
} by {
    real_complement_of_open_is_closed(open_lower_ray_set(lower))
    is_closed[Real](open_lower_ray_set(lower).c)
    closed_upper_ray_set_eq_complement_open_lower_ray_set(lower)
    closed_upper_ray_set(lower) = open_lower_ray_set(lower).c
    is_closed[Real](closed_upper_ray_set(lower))
}

/// Closed lower rays are closed sets.
theorem real_closed_lower_ray_set(upper: Real) {
    is_closed[Real](closed_lower_ray_set(upper))
} by {
    real_complement_of_open_is_closed(open_upper_ray_set(upper))
    is_closed[Real](open_upper_ray_set(upper).c)
    closed_lower_ray_set_eq_complement_open_upper_ray_set(upper)
    closed_lower_ray_set(upper) = open_upper_ray_set(upper).c
    is_closed[Real](closed_lower_ray_set(upper))
}

/// Open intervals of real numbers are open sets in the standard topology.
theorem open_interval_set_is_open(lower: Real, upper: Real) {
    Real.is_open(open_interval_set(lower, upper))
} by {
    real_open_upper_ray_set(lower)
    real_is_open(open_upper_ray_set(lower))
    Real.is_open(open_upper_ray_set(lower))
    real_open_lower_ray_set(upper)
    real_is_open(open_lower_ray_set(upper))
    Real.is_open(open_lower_ray_set(upper))
    Real.is_open(open_upper_ray_set(lower)) and Real.is_open(open_lower_ray_set(upper))
    open_inter[Real](open_upper_ray_set(lower), open_lower_ray_set(upper))
    Real.is_open(open_upper_ray_set(lower).intersection(open_lower_ray_set(upper)))
    open_interval_set_eq_ray_intersection(lower, upper)
    Real.is_open(open_interval_set(lower, upper))
}

/// Closed intervals of real numbers are closed sets.
theorem closed_interval_set_is_closed(lower: Real, upper: Real) {
    is_closed[Real](closed_interval_set(lower, upper))
} by {
    real_closed_upper_ray_set(lower)
    real_closed_lower_ray_set(upper)
    closed_inter[Real](closed_upper_ray_set(lower), closed_lower_ray_set(upper))
    is_closed[Real](closed_upper_ray_set(lower).intersection(closed_lower_ray_set(upper)))
    closed_interval_set_eq_ray_intersection(lower, upper)
    is_closed[Real](closed_interval_set(lower, upper))
}

/// A set unequal to the empty set has a member.
theorem nonempty_from_neq_empty(s: Set[Real]) {
    s != Set[Real].empty_set implies exists(x: Real) { s.contains(x) }
} by {
    if s != Set[Real].empty_set {
        if not exists(x: Real) { s.contains(x) } {
            forall(x: Real) {
                if s.contains(x) {
                    not exists(w: Real) { s.contains(w) }
                    false
                }
                not s.contains(x)
                empty_set_contains_eq[Real](x)
                Set[Real].empty_set.contains(x) = false
                s.contains(x) = Set[Real].empty_set.contains(x)
            }
            set_ext(s, Set[Real].empty_set)
            s = Set[Real].empty_set
            false
        }
        exists(x: Real) { s.contains(x) }
    }
}

/// A supremum is approximated from below: for every positive epsilon there is
/// a member of the set within epsilon of the supremum.
theorem supremum_approximation(s: Set[Real], c: Real, eps: Real) {
    is_set_supremum(s, c) and eps.is_positive implies exists(z: Real) {
        s.contains(z) and c - eps < z
    }
} by {
    if is_set_supremum(s, c) and eps.is_positive {
        is_set_supremum(s, c)
        eps.is_positive
        if not exists(z: Real) { s.contains(z) and c - eps < z } {
            forall(z: Real) {
                if s.contains(z) {
                    not (s.contains(z) and c - eps < z)
                    not (c - eps < z)
                    not_lt_imp_gte(c - eps, z)
                    c - eps >= z
                    z <= c - eps
                }
            }
            is_set_upper_bound(s, c - eps)
            is_set_supremum(s, c) = (is_set_upper_bound(s, c)
                and forall(b: Real) { is_set_upper_bound(s, b) implies c <= b })
            is_set_upper_bound(s, c - eps) implies c <= c - eps
            c <= c - eps
            sub_lt_self_of_pos(c, eps)
            c - eps < c
            lte_lt_trans(c, c - eps, c)
            c < c
            not_lt_self(c)
            false
        }
        exists(z: Real) { s.contains(z) and c - eps < z }
    }
}

/// Two disjoint open sets of which one contains a point strictly below a point
/// of the other cannot cover the real line.
theorem no_disconnection_with_order(u: Set[Real], v: Set[Real], x: Real, y: Real) {
    not (real_is_open(u) and real_is_open(v) and u.is_disjoint(v)
        and u.union(v) = Set[Real].universal_set and u.contains(x) and v.contains(y) and x < y)
} by {
    if real_is_open(u) and real_is_open(v) and u.is_disjoint(v)
        and u.union(v) = Set[Real].universal_set and u.contains(x) and v.contains(y) and x < y {
        real_is_open(u)
        real_is_open(v)
        u.is_disjoint(v)
        u.union(v) = Set[Real].universal_set
        u.contains(x)
        v.contains(y)
        x < y
        let s = u.intersection(closed_interval_set(x, y))
        lte_self(x)
        x <= x
        lt_imp_lte(x, y)
        x <= y
        closed_interval(x, y, x)
        closed_interval_set_contains_eq(x, y, x)
        closed_interval_set(x, y).contains(x)
        intersection_contains_intro(u, closed_interval_set(x, y), x)
        s.contains(x)
        is_nonempty(s)
        forall(z: Real) {
            if s.contains(z) {
                intersection_contains_eq(u, closed_interval_set(x, y), z)
                closed_interval_set(x, y).contains(z)
                closed_interval_set_contains_eq(x, y, z)
                closed_interval(x, y, z)
                z <= y
            }
        }
        is_set_upper_bound(s, y)
        has_upper_bound(s)
        completeness(s)
        let c: Real satisfy {
            is_set_supremum(s, c)
        }
        is_set_supremum(s, c)
        is_set_upper_bound(s, c)
        is_set_supremum(s, c) = (is_set_upper_bound(s, c)
            and forall(b: Real) { is_set_upper_bound(s, b) implies c <= b })
        forall(b: Real) { is_set_upper_bound(s, b) implies c <= b }
        is_set_upper_bound(s, y) implies c <= y
        c <= y
        is_set_upper_bound(s, c) = forall(z: Real) { s.contains(z) implies z <= c }
        s.contains(x) implies x <= c
        x <= c
        universal_set_contains_eq[Real](c)
        Set[Real].universal_set.contains(c)
        u.union(v).contains(c)
        union_contains_eq(u, v, c)
        u.contains(c) or v.contains(c)
        if u.contains(c) {
            real_open_is_interior_point(u, c)
            real_is_interior_point(u, c)
            real_is_interior_point(u, c) = (u.contains(c) and exists(eps: Real) {
                eps.is_positive and forall(w: Real) { w.is_close(c, eps) implies u.contains(w) }
            })
            exists(eps: Real) {
                eps.is_positive and forall(w: Real) { w.is_close(c, eps) implies u.contains(w) }
            }
            let eps0: Real satisfy {
                eps0.is_positive and forall(w: Real) { w.is_close(c, eps0) implies u.contains(w) }
            }
            eps0.is_positive
            if c = y {
                u.is_disjoint(v)
                not (u.contains(c) and v.contains(c))
                v.contains(c)
                u.contains(c) and v.contains(c)
                false
            }
            c != y
            if not (c < y) {
                not_lt_imp_gte(c, y)
                c >= y
                lte_both_ways_imp_eq(c, y)
                c = y
                false
            }
            c < y
            lt_imp_minus_pos(c, y)
            (y - c).is_positive
            eps_lt_half(eps0)
            let d0: Real satisfy {
                d0.is_positive and d0 + d0 < eps0
            }
            d0.is_positive
            positive_half_lt_bound(d0, eps0)
            d0 < eps0
            eps_lt_half(y - c)
            let d1: Real satisfy {
                d1.is_positive and d1 + d1 < y - c
            }
            d1.is_positive
            positive_half_lt_bound(d1, y - c)
            d1 < y - c
            let delta = d0.min(d1)
            lt_min_iff(Real.0, d0, d1)
            Real.0 < d0.min(d1) = (Real.0 < d0 and Real.0 < d1)
            Real.0 < d0
            Real.0 < d1
            Real.0 < d0.min(d1)
            Real.0 < delta
            min_lte_left(d0, d1)
            d0.min(d1) <= d0
            delta <= d0
            lte_lt_trans(delta, d0, eps0)
            delta < eps0
            min_lte_right(d0, d1)
            d0.min(d1) <= d1
            delta <= d1
            lte_lt_trans(delta, d1, y - c)
            delta < y - c
            lt_add_pos(c, delta)
            c < c + delta
            lt_add_right(delta, eps0, c)
            c + delta < c + eps0
            lt_add_right(delta, y - c, c)
            c + delta < c + (y - c)
            sub_cancels(y, c)
            y + c - c = y
            c + (y - c) = y
            c + delta < y
            sub_lt_self_of_pos(c, eps0)
            c - eps0 < c
            lt_trans(c - eps0, c, c + delta)
            c - eps0 < c + delta
            bounds_imp_close(c + delta, c, eps0)
            (c + delta).is_close(c, eps0)
            u.contains(c + delta)
            lt_imp_lte(c + delta, y)
            c + delta <= y
            lte_lt_trans(x, c, c + delta)
            x < c + delta
            lt_imp_lte(x, c + delta)
            x <= c + delta
            closed_interval(x, y, c + delta)
            closed_interval_set_contains_eq(x, y, c + delta)
            closed_interval_set(x, y).contains(c + delta)
            intersection_contains_intro(u, closed_interval_set(x, y), c + delta)
            s.contains(c + delta)
            is_set_upper_bound(s, c) = forall(z: Real) { s.contains(z) implies z <= c }
            s.contains(c + delta) implies (c + delta) <= c
            c + delta <= c
            lt_of_lt_of_lte(c, c + delta, c)
            c < c
            not_lt_self(c)
            false
        } else {
            v.contains(c)
            real_open_is_interior_point(v, c)
            real_is_interior_point(v, c)
            real_is_interior_point(v, c) = (v.contains(c) and exists(eps: Real) {
                eps.is_positive and forall(w: Real) { w.is_close(c, eps) implies v.contains(w) }
            })
            exists(eps: Real) {
                eps.is_positive and forall(w: Real) { w.is_close(c, eps) implies v.contains(w) }
            }
            let eps1: Real satisfy {
                eps1.is_positive and forall(w: Real) { w.is_close(c, eps1) implies v.contains(w) }
            }
            eps1.is_positive
            supremum_approximation(s, c, eps1)
            exists(z: Real) { s.contains(z) and c - eps1 < z }
            let z: Real satisfy { s.contains(z) and c - eps1 < z }
            s.contains(z)
            c - eps1 < z
            intersection_contains_eq(u, closed_interval_set(x, y), z)
            u.contains(z)
            is_set_upper_bound(s, c) = forall(w: Real) { s.contains(w) implies w <= c }
            s.contains(z) implies z <= c
            z <= c
            lt_add_pos(c, eps1)
            c < c + eps1
            lte_lt_trans(z, c, c + eps1)
            z < c + eps1
            bounds_imp_close(z, c, eps1)
            z.is_close(c, eps1)
            v.contains(z)
            u.is_disjoint(v)
            not (u.contains(z) and v.contains(z))
            u.contains(z) and v.contains(z)
            false
        }
    }
    not (real_is_open(u) and real_is_open(v) and u.is_disjoint(v) and u.union(v) = Set[Real].universal_set and u.contains(x) and v.contains(y) and x < y)
}

/// The real line is connected: it cannot be written as the union of two
/// disjoint nonempty open sets.
theorem real_line_is_connected {
    is_connected[Real](real_is_open)
} by {
    is_connected_iff[Real](real_is_open)
    if exists(u: Set[Real], v: Set[Real]) { is_disconnection[Real](real_is_open, u, v) } {
        let u: Set[Real] satisfy {
            exists(v: Set[Real]) { is_disconnection[Real](real_is_open, u, v) }
        }
        let v: Set[Real] satisfy {
            is_disconnection[Real](real_is_open, u, v)
        }
        is_disconnection_iff[Real](real_is_open, u, v)
        is_disconnection[Real](real_is_open, u, v) = (real_is_open(u) and real_is_open(v)
            and u != Set[Real].empty_set and v != Set[Real].empty_set
            and u.is_disjoint(v) and u.union(v) = Set[Real].universal_set)
        real_is_open(u) and real_is_open(v) and u != Set[Real].empty_set and v != Set[Real].empty_set and u.is_disjoint(v) and u.union(v) = Set[Real].universal_set
        real_is_open(u)
        real_is_open(v)
        u != Set[Real].empty_set
        v != Set[Real].empty_set
        u.is_disjoint(v)
        u.union(v) = Set[Real].universal_set
        nonempty_from_neq_empty(u)
        exists(x: Real) { u.contains(x) }
        let x: Real satisfy { u.contains(x) }
        nonempty_from_neq_empty(v)
        exists(y: Real) { v.contains(y) }
        let y: Real satisfy { v.contains(y) }
        u.contains(x)
        v.contains(y)
        if x = y {
            u.is_disjoint(v)
            not (u.contains(x) and v.contains(x))
            v.contains(x)
            u.contains(x) and v.contains(x)
            false
        }
        x != y
        lt_or_lte(x, y)
        x < y or y <= x
        if x < y {
            real_is_open(u) and real_is_open(v) and u.is_disjoint(v) and u.union(v) = Set[Real].universal_set and u.contains(x) and v.contains(y) and x < y
            no_disconnection_with_order(u, v, x, y)
            false
        }
        y <= x
        lt_or_lte(y, x)
        y < x or x <= y
        if y < x {
            union_comm(u, v)
            u.union(v) = v.union(u)
            v.union(u) = Set[Real].universal_set
            forall(z: Real) {
                if v.contains(z) and u.contains(z) {
                    u.is_disjoint(v)
                    not (u.contains(z) and v.contains(z))
                    u.contains(z) and v.contains(z)
                    false
                }
            }
            v.is_disjoint(u)
            real_is_open(v) and real_is_open(u) and v.is_disjoint(u) and v.union(u) = Set[Real].universal_set and v.contains(y) and u.contains(x) and y < x
            no_disconnection_with_order(v, u, y, x)
            false
        }
        x <= y
        lte_both_ways_imp_eq(x, y)
        x = y
        false
    }
    not exists(u: Set[Real], v: Set[Real]) {
        is_disconnection[Real](real_is_open, u, v)
    }
}

/// A degenerate closed interval is contained in any set containing its point.
theorem singleton_interval_subset(lower: Real, u: Set[Real]) {
    u.contains(lower) implies closed_interval_set(lower, lower).subset(u)
} by {
    if u.contains(lower) {
        forall(x: Real) {
            if closed_interval_set(lower, lower).contains(x) {
                closed_interval_set_contains_eq(lower, lower, x)
                closed_interval(lower, lower, x)
                lower <= x
                x <= lower
                lte_both_ways_imp_eq(lower, x)
                x = lower
                u.contains(x)
            }
        }
    }
}

/// A closed interval with reversed endpoints is empty.
theorem reversed_interval_set_is_empty(lower: Real, upper: Real) {
    not lower <= upper implies closed_interval_set(lower, upper) = Set[Real].empty_set
} by {
    if not lower <= upper {
        forall(x: Real) {
            if closed_interval_set(lower, upper).contains(x) {
                closed_interval_set_contains_eq(lower, upper, x)
                closed_interval(lower, upper, x)
                lower <= x
                x <= upper
                lte_trans(lower, x, upper)
                lower <= upper
                false
            }
            not closed_interval_set(lower, upper).contains(x)
            empty_set_contains_eq[Real](x)
            Set[Real].empty_set.contains(x) = false
            closed_interval_set(lower, upper).contains(x) = Set[Real].empty_set.contains(x)
        }
        set_ext(closed_interval_set(lower, upper), Set[Real].empty_set)
        closed_interval_set(lower, upper) = Set[Real].empty_set
    }
}

/// A closed interval is contained in the union of two intervals splitting it.
theorem interval_subset_union_intervals(a: Real, x: Real, y: Real) {
    closed_interval_set(a, y).subset(closed_interval_set(a, x).union(closed_interval_set(x, y)))
} by {
    forall(t: Real) {
        if closed_interval_set(a, y).contains(t) {
            closed_interval_set_contains_eq(a, y, t)
            closed_interval(a, y, t)
            a <= t
            t <= y
            if t <= x {
                closed_interval(a, x, t)
                closed_interval_set_contains_eq(a, x, t)
                closed_interval_set(a, x).contains(t)
                union_contains_eq(closed_interval_set(a, x), closed_interval_set(x, y), t)
                closed_interval_set(a, x).union(closed_interval_set(x, y)).contains(t)
            } else {
                not_lte_imp_gt(t, x)
                t > x
                x < t
                lt_imp_lte(x, t)
                x <= t
                closed_interval(x, y, t)
                closed_interval_set_contains_eq(x, y, t)
                closed_interval_set(x, y).contains(t)
                union_contains_eq(closed_interval_set(a, x), closed_interval_set(x, y), t)
                closed_interval_set(a, x).union(closed_interval_set(x, y)).contains(t)
            }
        }
    }
}

/// A union of subsets of two sets is a subset of their union.
theorem union_subset_of_union(a: Set[Real], b: Set[Real], u: Set[Real], v: Set[Real]) {
    a.subset(u) and b.subset(v) implies a.union(b).subset(u.union(v))
} by {
    if a.subset(u) and b.subset(v) {
        forall(x: Real) {
            if a.union(b).contains(x) {
                union_contains_eq(a, b, x)
                if a.contains(x) {
                    a.subset(u) implies (a.contains(x) implies u.contains(x))
                    u.contains(x)
                    union_contains_eq(u, v, x)
                    u.union(v).contains(x)
                } else {
                    b.contains(x)
                    b.subset(v) implies (b.contains(x) implies v.contains(x))
                    v.contains(x)
                    union_contains_eq(u, v, x)
                    u.union(v).contains(x)
                }
            }
        }
    }
}

/// True if the prefix `[a, x]` is covered by finitely many members of the
/// family `c`.
define finitely_coverable(c: Set[Real] -> Bool, a: Real, x: Real) -> Bool {
    exists(items: List[Set[Real]]) {
        is_subfamily[Real](c, items) and closed_interval_set(a, x).subset(list_union_of_sets[Real](items))
    }
}

/// True if `x` is a prefix of `[lower, upper]` whose interval `[lower, x]`
/// is covered by finitely many members of the family `c`.
define prefix_member(c: Set[Real] -> Bool, lower: Real, upper: Real, x: Real) -> Bool {
    lower <= x and x <= upper and finitely_coverable(c, lower, x)
}

/// The set of finitely-coverable prefixes of the interval `[lower, upper]`.
define prefix_set(c: Set[Real] -> Bool, lower: Real, upper: Real) -> Set[Real] {
    Set[Real].new(prefix_member(c, lower, upper))
}

/// Membership in the prefix set is the prefix-membership predicate.
theorem prefix_set_contains_eq(c: Set[Real] -> Bool, lower: Real, upper: Real, x: Real) {
    prefix_set(c, lower, upper).contains(x) =
        (lower <= x and x <= upper and finitely_coverable(c, lower, x))
}

/// Extending a finite subcover of `[a, x]` by an open set covering `[x, y]`
/// gives a finite subcover of `[a, y]`.
theorem extend_finite_subcover(c: Set[Real] -> Bool, a: Real, x: Real, y: Real,
    u: Set[Real], items: List[Set[Real]]) {
    finitely_coverable(c, a, x) and c(u) and a <= x and x <= y
        and closed_interval_set(x, y).subset(u)
    implies finitely_coverable(c, a, y)
} by {
    if finitely_coverable(c, a, x) and c(u) and a <= x and x <= y
        and closed_interval_set(x, y).subset(u) {
        finitely_coverable(c, a, x)
        c(u)
        a <= x
        x <= y
        closed_interval_set(x, y).subset(u)
        finitely_coverable(c, a, x) = exists(items0: List[Set[Real]]) {
            is_subfamily[Real](c, items0) and closed_interval_set(a, x).subset(list_union_of_sets[Real](items0))
        }
        exists(items0: List[Set[Real]]) {
            is_subfamily[Real](c, items0) and closed_interval_set(a, x).subset(list_union_of_sets[Real](items0))
        }
        let items0: List[Set[Real]] satisfy {
            is_subfamily[Real](c, items0) and closed_interval_set(a, x).subset(list_union_of_sets[Real](items0))
        }
        is_subfamily[Real](c, items0)
        closed_interval_set(a, x).subset(list_union_of_sets[Real](items0))
        interval_subset_union_intervals(a, x, y)
        closed_interval_set(a, y).subset(closed_interval_set(a, x).union(closed_interval_set(x, y)))
        union_subset_of_union(closed_interval_set(a, x), closed_interval_set(x, y),
            list_union_of_sets[Real](items0), u)
        closed_interval_set(a, x).union(closed_interval_set(x, y)).subset(list_union_of_sets[Real](items0).union(u))
        subset_trans(closed_interval_set(a, y), closed_interval_set(a, x).union(closed_interval_set(x, y)),
            list_union_of_sets[Real](items0).union(u))
        closed_interval_set(a, y).subset(list_union_of_sets[Real](items0).union(u))
        let items1: List[Set[Real]] = List.cons(u, items0)
        list_union_of_sets_cons[Real](u, items0)
        list_union_of_sets[Real](items1) = u.union(list_union_of_sets[Real](items0))
        union_comm(u, list_union_of_sets[Real](items0))
        u.union(list_union_of_sets[Real](items0)) = list_union_of_sets[Real](items0).union(u)
        list_union_of_sets[Real](items1) = list_union_of_sets[Real](items0).union(u)
        closed_interval_set(a, y).subset(list_union_of_sets[Real](items1))
        is_subfamily_cons_mpr[Real](c, u, items0)
        c(u) and is_subfamily[Real](c, items0) implies is_subfamily[Real](c, List.cons(u, items0))
        is_subfamily[Real](c, items1)
        is_subfamily[Real](c, items1) and closed_interval_set(a, y).subset(list_union_of_sets[Real](items1))
        exists(items2: List[Set[Real]]) {
            is_subfamily[Real](c, items2) and closed_interval_set(a, y).subset(list_union_of_sets[Real](items2))
        }
        finitely_coverable(c, a, y)
    }
}

/// A ball around `m` covers the interval `[s, m]` when `s` lies within the
/// ball and below `m`.
theorem interval_after_ball_subset(m: Real, eps: Real, s: Real, u: Set[Real]) {
    m - eps < s and s <= m and eps.is_positive
        and (forall(w: Real) { w.is_close(m, eps) implies u.contains(w) })
    implies closed_interval_set(s, m).subset(u)
} by {
    if m - eps < s and s <= m and eps.is_positive
        and (forall(w: Real) { w.is_close(m, eps) implies u.contains(w) }) {
        m - eps < s
        s <= m
        eps.is_positive
        forall(w: Real) { w.is_close(m, eps) implies u.contains(w) }
        forall(t: Real) {
            if closed_interval_set(s, m).contains(t) {
                closed_interval_set_lower_le(s, m, t)
                s <= t
                closed_interval_set_le_upper(s, m, t)
                t <= m
                m - eps < s and s <= t
                lt_lte_trans(m - eps, s, t)
                m - eps < t
                lt_add_pos(m, eps)
                m < m + eps
                t <= m and m < m + eps
                lte_lt_trans(t, m, m + eps)
                t < m + eps
                bounds_imp_close(t, m, eps)
                t.is_close(m, eps)
                u.contains(t)
            }
        }
    }
}

/// A ball around `m` covers the interval `[m, y]` when `y` lies within the
/// ball and above `m`.
theorem interval_before_ball_subset(m: Real, eps: Real, y: Real, u: Set[Real]) {
    y < m + eps and m <= y and eps.is_positive
        and (forall(w: Real) { w.is_close(m, eps) implies u.contains(w) })
    implies closed_interval_set(m, y).subset(u)
} by {
    if y < m + eps and m <= y and eps.is_positive
        and (forall(w: Real) { w.is_close(m, eps) implies u.contains(w) }) {
        y < m + eps
        m <= y
        eps.is_positive
        forall(w: Real) { w.is_close(m, eps) implies u.contains(w) }
        forall(t: Real) {
            if closed_interval_set(m, y).contains(t) {
                closed_interval_set_lower_le(m, y, t)
                m <= t
                closed_interval_set_le_upper(m, y, t)
                t <= y
                sub_lt_self_of_pos(m, eps)
                m - eps < m
                m - eps < m and m <= t
                lt_lte_trans(m - eps, m, t)
                m - eps < t
                t <= y and y < m + eps
                lte_lt_trans(t, y, m + eps)
                t < m + eps
                bounds_imp_close(t, m, eps)
                t.is_close(m, eps)
                u.contains(t)
            }
        }
    }
}

/// A closed interval with ordered endpoints is compact: every open cover has
/// a finite subcover (Heine-Borel).
theorem heine_borel_closed_interval(lower: Real, upper: Real) {
    is_compact[Real](closed_interval_set(lower, upper))
} by {
    if lower <= upper {
        is_compact_iff[Real](closed_interval_set(lower, upper))
        forall(c: Set[Real] -> Bool) {
            if is_open_cover[Real](c, closed_interval_set(lower, upper)) {
                is_open_cover_iff[Real](c, closed_interval_set(lower, upper))
                is_open_cover[Real](c, closed_interval_set(lower, upper)) =
                    ((forall(u: Set[Real]) { c(u) implies Real.is_open(u) })
                        and closed_interval_set(lower, upper).subset(big_union(c)))
                forall(u: Set[Real]) { c(u) implies Real.is_open(u) }
                closed_interval_set(lower, upper).subset(big_union(c))
                let s = prefix_set(c, lower, upper)
                lte_self(lower)
                lower <= lower
                lower <= upper
                // the lower endpoint lies in the big union of the cover
                closed_interval_set_contains_eq(lower, upper, lower)
                closed_interval(lower, upper, lower)
                closed_interval_set(lower, upper).contains(lower)
                subset_contains(closed_interval_set(lower, upper), big_union(c), lower)
                big_union(c).contains(lower)
                big_union_contains_eq(c, lower)
                big_union(c).contains(lower) = exists(u0: Set[Real]) { c(u0) and u0.contains(lower) }
                exists(u0: Set[Real]) { c(u0) and u0.contains(lower) }
                let u0: Set[Real] satisfy { c(u0) and u0.contains(lower) }
                c(u0)
                u0.contains(lower)
                singleton_interval_subset(lower, u0)
                closed_interval_set(lower, lower).subset(u0)
                is_subfamily_nil[Real](c)
                is_subfamily[Real](c, List.nil[Set[Real]])
                let base_items: List[Set[Real]] = List.cons(u0, List.nil[Set[Real]])
                is_subfamily_cons_mpr[Real](c, u0, List.nil[Set[Real]])
                c(u0) and is_subfamily[Real](c, List.nil[Set[Real]]) implies is_subfamily[Real](c, List.cons(u0, List.nil[Set[Real]]))
                is_subfamily[Real](c, base_items)
                list_union_of_sets_cons[Real](u0, List.nil[Set[Real]])
                list_union_of_sets[Real](base_items) = u0.union(list_union_of_sets[Real](List.nil[Set[Real]]))
                list_union_of_sets_nil[Real]
                list_union_of_sets[Real](List.nil[Set[Real]]) = Set[Real].empty_set
                u0.union(Set[Real].empty_set) = u0
                list_union_of_sets[Real](base_items) = u0
                closed_interval_set(lower, lower).subset(list_union_of_sets[Real](base_items))
                is_subfamily[Real](c, base_items) and closed_interval_set(lower, lower).subset(list_union_of_sets[Real](base_items))
                exists(items0: List[Set[Real]]) {
                    is_subfamily[Real](c, items0) and closed_interval_set(lower, lower).subset(list_union_of_sets[Real](items0))
                }
                finitely_coverable(c, lower, lower)
                prefix_set_contains_eq(c, lower, upper, lower)
                prefix_set(c, lower, upper).contains(lower) = (lower <= lower and lower <= upper and finitely_coverable(c, lower, lower))
                prefix_set(c, lower, upper).contains(lower)
                is_nonempty(prefix_set(c, lower, upper))
                // the upper endpoint is an upper bound of the prefix set
                forall(x: Real) {
                    if prefix_set(c, lower, upper).contains(x) {
                        prefix_set_contains_eq(c, lower, upper, x)
                        prefix_set(c, lower, upper).contains(x) = (lower <= x and x <= upper and finitely_coverable(c, lower, x))
                        lower <= x and x <= upper and finitely_coverable(c, lower, x)
                        x <= upper
                    }
                }
                is_set_upper_bound(prefix_set(c, lower, upper), upper)
                has_upper_bound(prefix_set(c, lower, upper))
                completeness(prefix_set(c, lower, upper))
                let m: Real satisfy {
                    is_set_supremum(prefix_set(c, lower, upper), m)
                }
                is_set_supremum(prefix_set(c, lower, upper), m)
                is_set_upper_bound(prefix_set(c, lower, upper), m)
                // m lies between the endpoints
                is_set_upper_bound(prefix_set(c, lower, upper), m) = forall(x: Real) { prefix_set(c, lower, upper).contains(x) implies x <= m }
                prefix_set(c, lower, upper).contains(lower) implies lower <= m
                lower <= m
                is_set_supremum(prefix_set(c, lower, upper), m) = (is_set_upper_bound(prefix_set(c, lower, upper), m)
                    and forall(b: Real) { is_set_upper_bound(prefix_set(c, lower, upper), b) implies m <= b })
                forall(b: Real) { is_set_upper_bound(prefix_set(c, lower, upper), b) implies m <= b }
                is_set_upper_bound(prefix_set(c, lower, upper), upper) implies m <= upper
                m <= upper
                // m lies in the big union of the cover, witnessed by an open set
                closed_interval_set_contains_eq(lower, upper, m)
                closed_interval(lower, upper, m)
                closed_interval_set(lower, upper).contains(m)
                subset_contains(closed_interval_set(lower, upper), big_union(c), m)
                big_union(c).contains(m)
                big_union_contains_eq(c, m)
                exists(um: Set[Real]) { c(um) and um.contains(m) }
                let um: Set[Real] satisfy { c(um) and um.contains(m) }
                c(um)
                um.contains(m)
                c(um) implies Real.is_open(um)
                Real.is_open(um)
                real_open_is_interior_point(um, m)
                real_is_interior_point(um, m)
                real_is_interior_point(um, m) = (um.contains(m) and exists(eps: Real) {
                    eps.is_positive and forall(w: Real) { w.is_close(m, eps) implies um.contains(w) }
                })
                exists(eps: Real) {
                    eps.is_positive and forall(w: Real) { w.is_close(m, eps) implies um.contains(w) }
                }
                let eps: Real satisfy {
                    eps.is_positive and forall(w: Real) { w.is_close(m, eps) implies um.contains(w) }
                }
                eps.is_positive
                forall(w: Real) { w.is_close(m, eps) implies um.contains(w) }
                // m itself is finitely coverable
                if lower < m {
                    supremum_approximation(prefix_set(c, lower, upper), m, eps)
                    exists(x: Real) { prefix_set(c, lower, upper).contains(x) and m - eps < x }
                    let sm: Real satisfy { prefix_set(c, lower, upper).contains(sm) and m - eps < sm }
                    prefix_set(c, lower, upper).contains(sm)
                    m - eps < sm
                    is_set_upper_bound(prefix_set(c, lower, upper), m) = forall(x: Real) { prefix_set(c, lower, upper).contains(x) implies x <= m }
                    prefix_set(c, lower, upper).contains(sm) implies sm <= m
                    sm <= m
                    prefix_set_contains_eq(c, lower, upper, sm)
                    prefix_set(c, lower, upper).contains(sm) = (lower <= sm and sm <= upper and finitely_coverable(c, lower, sm))
                    lower <= sm and sm <= upper and finitely_coverable(c, lower, sm)
                    finitely_coverable(c, lower, sm)
                    interval_after_ball_subset(m, eps, sm, um)
                    closed_interval_set(sm, m).subset(um)
                    extend_finite_subcover(c, lower, sm, m, um, List.nil[Set[Real]])
                    finitely_coverable(c, lower, m)
                } else {
                    not lower < m
                    not_lt_imp_gte(lower, m)
                    lower >= m
                    lte_both_ways_imp_eq(lower, m)
                    lower = m
                    um.contains(m)
                    um.contains(lower)
                    singleton_interval_subset(lower, um)
                    closed_interval_set(lower, lower).subset(um)
                    is_subfamily_nil[Real](c)
                    is_subfamily[Real](c, List.nil[Set[Real]])
                    let base_items2: List[Set[Real]] = List.cons(um, List.nil[Set[Real]])
                    is_subfamily_cons_mpr[Real](c, um, List.nil[Set[Real]])
                    is_subfamily[Real](c, base_items2)
                    list_union_of_sets_cons[Real](um, List.nil[Set[Real]])
                    list_union_of_sets[Real](base_items2) = um.union(list_union_of_sets[Real](List.nil[Set[Real]]))
                    list_union_of_sets_nil[Real]
                    list_union_of_sets[Real](List.nil[Set[Real]]) = Set[Real].empty_set
                    um.union(Set[Real].empty_set) = um
                    list_union_of_sets[Real](base_items2) = um
                    closed_interval_set(lower, lower).subset(list_union_of_sets[Real](base_items2))
                    is_subfamily[Real](c, base_items2) and closed_interval_set(lower, lower).subset(list_union_of_sets[Real](base_items2))
                    exists(items0: List[Set[Real]]) {
                        is_subfamily[Real](c, items0) and closed_interval_set(lower, lower).subset(list_union_of_sets[Real](items0))
                    }
                    finitely_coverable(c, lower, lower)
                    lower = m
                    finitely_coverable(c, lower, m)
                }
                finitely_coverable(c, lower, m)
                // the supremum must be the upper endpoint
                if m < upper {
                    eps_lt_half(eps)
                    let d0: Real satisfy { d0.is_positive and d0 + d0 < eps }
                    d0.is_positive
                    positive_half_lt_bound(d0, eps)
                    d0 < eps
                    lt_imp_minus_pos(m, upper)
                    (upper - m).is_positive
                    eps_lt_half(upper - m)
                    let d1: Real satisfy { d1.is_positive and d1 + d1 < upper - m }
                    d1.is_positive
                    positive_half_lt_bound(d1, upper - m)
                    d1 < upper - m
                    let delta = d0.min(d1)
                    lt_min_iff(Real.0, d0, d1)
                    Real.0 < d0.min(d1) = (Real.0 < d0 and Real.0 < d1)
                    Real.0 < d0
                    Real.0 < d1
                    Real.0 < d0.min(d1)
                    Real.0 < delta
                    min_lte_left(d0, d1)
                    delta <= d0
                    lte_lt_trans(delta, d0, eps)
                    delta < eps
                    min_lte_right(d0, d1)
                    delta <= d1
                    lte_lt_trans(delta, d1, upper - m)
                    delta < upper - m
                    lt_add_pos(m, delta)
                    m < m + delta
                    lt_add_right(delta, upper - m, m)
                    m + delta < m + (upper - m)
                    sub_cancels(upper, m)
                    upper + m - m = upper
                    m + (upper - m) = upper
                    m + delta < upper
                    lt_imp_lte(m + delta, upper)
                    m + delta <= upper
                    lt_imp_lte(m, m + delta)
                    m <= m + delta
                    lt_add_right(delta, eps, m)
                    m + delta < m + eps
                    interval_before_ball_subset(m, eps, m + delta, um)
                    closed_interval_set(m, m + delta).subset(um)
                    extend_finite_subcover(c, lower, m, m + delta, um, List.nil[Set[Real]])
                    finitely_coverable(c, lower, m + delta)
                    lte_lt_trans(lower, m, m + delta)
                    lower < m + delta
                    lt_imp_lte(lower, m + delta)
                    lower <= m + delta
                    prefix_set_contains_eq(c, lower, upper, m + delta)
                    prefix_set(c, lower, upper).contains(m + delta) = (lower <= m + delta and m + delta <= upper and finitely_coverable(c, lower, m + delta))
                    prefix_set(c, lower, upper).contains(m + delta)
                    is_set_upper_bound(prefix_set(c, lower, upper), m) = forall(x: Real) { prefix_set(c, lower, upper).contains(x) implies x <= m }
                    prefix_set(c, lower, upper).contains(m + delta) implies m + delta <= m
                    m + delta <= m
                    lt_of_lt_of_lte(m, m + delta, m)
                    m < m
                    not_lt_self(m)
                    false
                }
                not m < upper
                not_lt_imp_gte(m, upper)
                m >= upper
                lte_both_ways_imp_eq(m, upper)
                m = upper
                finitely_coverable(c, lower, m) = exists(items: List[Set[Real]]) {
                    is_subfamily[Real](c, items) and closed_interval_set(lower, m).subset(list_union_of_sets[Real](items))
                }
                exists(items: List[Set[Real]]) {
                    is_subfamily[Real](c, items) and closed_interval_set(lower, m).subset(list_union_of_sets[Real](items))
                }
                let items_final: List[Set[Real]] satisfy {
                    is_subfamily[Real](c, items_final) and closed_interval_set(lower, m).subset(list_union_of_sets[Real](items_final))
                }
                is_subfamily[Real](c, items_final)
                closed_interval_set(lower, m).subset(list_union_of_sets[Real](items_final))
                m = upper
                closed_interval_set(lower, upper).subset(list_union_of_sets[Real](items_final))
                is_subfamily[Real](c, items_final) and closed_interval_set(lower, upper).subset(list_union_of_sets[Real](items_final))
                exists(items2: List[Set[Real]]) {
                    is_subfamily[Real](c, items2) and closed_interval_set(lower, upper).subset(list_union_of_sets[Real](items2))
                }
            }
        }
        forall(c: Set[Real] -> Bool) {
            is_open_cover[Real](c, closed_interval_set(lower, upper)) implies exists(items: List[Set[Real]]) {
                is_subfamily[Real](c, items) and closed_interval_set(lower, upper).subset(list_union_of_sets[Real](items))
            }
        }
        is_compact_iff[Real](closed_interval_set(lower, upper))
        if not is_compact[Real](closed_interval_set(lower, upper)) {
            let w: (Set[Real] -> Bool) satisfy {
                is_open_cover[Real](w, closed_interval_set(lower, upper)) and not exists(items: List[Set[Real]]) {
                    is_subfamily[Real](w, items) and closed_interval_set(lower, upper).subset(list_union_of_sets[Real](items))
                }
            }
            is_open_cover[Real](w, closed_interval_set(lower, upper))
            not exists(items: List[Set[Real]]) {
                is_subfamily[Real](w, items) and closed_interval_set(lower, upper).subset(list_union_of_sets[Real](items))
            }
            forall(c: Set[Real] -> Bool) {
                is_open_cover[Real](c, closed_interval_set(lower, upper)) implies exists(items: List[Set[Real]]) {
                    is_subfamily[Real](c, items) and closed_interval_set(lower, upper).subset(list_union_of_sets[Real](items))
                }
            }
            is_open_cover[Real](w, closed_interval_set(lower, upper)) implies exists(items: List[Set[Real]]) {
                is_subfamily[Real](w, items) and closed_interval_set(lower, upper).subset(list_union_of_sets[Real](items))
            }
            exists(items: List[Set[Real]]) {
                is_subfamily[Real](w, items) and closed_interval_set(lower, upper).subset(list_union_of_sets[Real](items))
            }
            false
        }
        is_compact[Real](closed_interval_set(lower, upper))
    } else {
        not lower <= upper
        reversed_interval_set_is_empty(lower, upper)
        closed_interval_set(lower, upper) = Set[Real].empty_set
        empty_set_is_compact[Real]
        is_compact[Real](Set[Real].empty_set)
        is_compact[Real](closed_interval_set(lower, upper))
    }
}
