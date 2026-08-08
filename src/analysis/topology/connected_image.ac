from data.basic.functions import is_surjective_fn
from data.basic.set import Set, set_preimage, set_preimage_compl, set_preimage_empty,
    set_preimage_eq_imp_eq_of_surjective, set_preimage_contains_eq,
    set_preimage_union, set_preimage_universal
from analysis.topology.topological_space import TopologicalSpace, is_continuous,
    is_clopen, is_clopen_imp_open, is_clopen_intro,
    is_disconnection, is_disconnection_intro, is_connected, is_connected_iff
from analysis.topology.connectedness import disconnection_imp_open_left, disconnection_imp_open_right,
    disconnection_imp_fields
from analysis.topology.continuous_preimage import continuous_open_preimage

/// The preimage of a clopen set under a continuous map is clopen.
theorem continuous_preimage_clopen[X: TopologicalSpace, Y: TopologicalSpace](
        f: X -> Y, s: Set[Y]) {
    is_continuous(f) and is_clopen[Y](Y.is_open, s)
        implies is_clopen[X](X.is_open, set_preimage(f, s))
} by {
    if is_continuous(f) and is_clopen[Y](Y.is_open, s) {
        is_clopen_imp_open[Y](Y.is_open, s)
        continuous_open_preimage[X, Y](f, s)
        continuous_open_preimage[X, Y](f, s.c)
        set_preimage_compl[X, Y](f, s)
        X.is_open(set_preimage(f, s).c)
        is_clopen_intro[X](X.is_open, set_preimage(f, s))
        is_clopen[X](X.is_open, set_preimage(f, s))
    }
}

/// A nonempty target set has nonempty preimage under a surjective map.
theorem surjective_preimage_nonempty[X, Y](f: X -> Y, s: Set[Y]) {
    is_surjective_fn(f) and s != Set[Y].empty_set implies
        set_preimage(f, s) != Set[X].empty_set
} by {
    if is_surjective_fn(f) and s != Set[Y].empty_set {
        if set_preimage(f, s) = Set[X].empty_set {
            set_preimage_empty[X, Y](f)
            set_preimage_eq_imp_eq_of_surjective[X, Y](s, Set[Y].empty_set, f)
            false
        }
        set_preimage(f, s) != Set[X].empty_set
    }
}

/// Preimage preserves disjointness.
theorem preimage_disjoint_of_disjoint[X, Y](f: X -> Y, u: Set[Y], v: Set[Y]) {
    u.is_disjoint(v) implies set_preimage(f, u).is_disjoint(set_preimage(f, v))
} by {
    if u.is_disjoint(v) {
        forall(x: X) {
            if set_preimage(f, u).contains(x) and set_preimage(f, v).contains(x) {
                set_preimage_contains_eq[X, Y](f, u, x)
                set_preimage_contains_eq[X, Y](f, v, x)
                v.contains(f(x))
                u.is_disjoint(v) = forall(y: Y) {
                    not (u.contains(y) and v.contains(y))
                }
                false
            }
            not (set_preimage(f, u).contains(x) and set_preimage(f, v).contains(x))
        }
    }
}

/// Preimage preserves a binary cover of the universe.
theorem preimage_union_universal_of_union_universal[X, Y](f: X -> Y,
        u: Set[Y], v: Set[Y]) {
    u.union(v) = Set[Y].universal_set implies
        set_preimage(f, u).union(set_preimage(f, v)) = Set[X].universal_set
} by {
    if u.union(v) = Set[Y].universal_set {
        set_preimage_union[X, Y](f, u, v)
        set_preimage_universal[X, Y](f)
        set_preimage(f, u).union(set_preimage(f, v)) = Set[X].universal_set
    }
}

/// The preimage of a disconnection under a continuous surjection is a disconnection.
theorem continuous_surjective_preimage_disconnection[X: TopologicalSpace, Y: TopologicalSpace](
        f: X -> Y, u: Set[Y], v: Set[Y]) {
    is_continuous(f) and is_surjective_fn(f)
        and is_disconnection[Y](Y.is_open, u, v)
        implies is_disconnection[X](X.is_open, set_preimage(f, u), set_preimage(f, v))
} by {
    if is_continuous(f) and is_surjective_fn(f)
        and is_disconnection[Y](Y.is_open, u, v) {
        disconnection_imp_open_left[Y](Y.is_open, u, v)
        disconnection_imp_open_right[Y](Y.is_open, u, v)
        continuous_open_preimage[X, Y](f, u)
        continuous_open_preimage[X, Y](f, v)
        disconnection_imp_fields[Y](Y.is_open, u, v)
        surjective_preimage_nonempty[X, Y](f, u)
        surjective_preimage_nonempty[X, Y](f, v)
        set_preimage(f, v) != Set[X].empty_set
        preimage_disjoint_of_disjoint[X, Y](f, u, v)
        preimage_union_universal_of_union_universal[X, Y](f, u, v)
        is_disconnection_intro[X](X.is_open, set_preimage(f, u), set_preimage(f, v))
        is_disconnection[X](X.is_open, set_preimage(f, u), set_preimage(f, v))
    }
}

/// A continuous surjection from a connected space has connected codomain.
theorem surjective_continuous_connected_image[X: TopologicalSpace, Y: TopologicalSpace](
        f: X -> Y) {
    is_continuous(f) and is_surjective_fn(f) and is_connected[X](X.is_open)
        implies is_connected[Y](Y.is_open)
} by {
    if is_continuous(f) and is_surjective_fn(f) and is_connected[X](X.is_open) {
        is_connected_iff[Y](Y.is_open)
        if exists(u: Set[Y], v: Set[Y]) { is_disconnection[Y](Y.is_open, u, v) } {
            let u: Set[Y] satisfy {
                exists(v: Set[Y]) { is_disconnection[Y](Y.is_open, u, v) }
            }
            let v: Set[Y] satisfy {
                is_disconnection[Y](Y.is_open, u, v)
            }
            continuous_surjective_preimage_disconnection[X, Y](f, u, v)
            is_connected_iff[X](X.is_open)
            false
        }
        is_connected[Y](Y.is_open)
    }
}
