from list import List, filter_contains_and
from data.basic.set import Set, set_image, subset_contains, subset_contains_eq,
    empty_set_contains_eq, union_contains_eq, compl_contains_eq
from analysis.topology.topological_space import TopologicalSpace, big_union, big_union_contains_eq,
    big_union_contains_of_member, is_closed, is_continuous, is_open_cover,
    is_open_cover_iff, is_compact, is_subfamily, is_subfamily_cons_mp,
    is_subfamily_cons_mpr, list_union_of_sets, list_union_of_sets_nil,
    list_union_of_sets_cons
from analysis.topology.compact_union import compact_open_cover_has_finite_subcover, compact_union
from analysis.topology.compact_image import continuous_image_compact

/// The family obtained from a cover of `s` by adjoining the complement of `s`.
define cover_adjoin_complement[T](s: Set[T], c: Set[T] -> Bool, u: Set[T]) -> Bool {
    c(u) or u = s.c
}

/// Adjoining the complement of a closed set to an open family again gives an open family.
theorem cover_adjoin_complement_open_sets[T: TopologicalSpace](s: Set[T], c: Set[T] -> Bool) {
    is_closed[T](s) and (forall(u: Set[T]) { c(u) implies T.is_open(u) })
        implies forall(u: Set[T]) { cover_adjoin_complement[T](s, c, u) implies T.is_open(u) }
} by {
    if is_closed[T](s) and forall(u: Set[T]) { c(u) implies T.is_open(u) } {
        forall(u: Set[T]) {
            if cover_adjoin_complement[T](s, c, u) {
                if c(u) {
                    T.is_open(u)
                } else {
                    u = s.c
                    T.is_open(u)
                }
            }
        }
    }
}

/// An open cover of a closed set extends to any ambient set after adjoining the complement.
theorem cover_adjoin_complement_is_open_cover_of_superset[T: TopologicalSpace](
    s: Set[T],
    k: Set[T],
    c: Set[T] -> Bool
) {
    is_closed[T](s) and is_open_cover[T](c, s)
        implies is_open_cover[T](cover_adjoin_complement[T](s, c), k)
} by {
    if is_closed[T](s) and is_open_cover[T](c, s) {
        let d: Set[T] -> Bool = cover_adjoin_complement[T](s, c)
        is_open_cover_iff[T](c, s)
        forall(u: Set[T]) { c(u) implies T.is_open(u) }
        forall(u: Set[T]) {
            if d(u) {
                cover_adjoin_complement[T](s, c, u) = (c(u) or u = s.c)
                if c(u) {
                    T.is_open(u)
                } else {
                    u = s.c
                    T.is_open(u)
                }
            }
        }
        forall(x: T) {
            if k.contains(x) {
                if s.contains(x) {
                    subset_contains[T](s, big_union[T](c), x)
                    big_union_contains_eq[T](c, x)
                    let u: Set[T] satisfy {
                        c(u) and u.contains(x)
                    }
                    cover_adjoin_complement[T](s, c, u) = (c(u) or u = s.c)
                    big_union_contains_of_member[T](d, u, x)
                    big_union[T](d).contains(x)
                } else {
                    compl_contains_eq[T](s, x)
                    d(s.c)
                    big_union_contains_of_member[T](d, s.c, x)
                    big_union[T](d).contains(x)
                }
            }
        }
        subset_contains_eq[T](k, big_union[T](d))
        k.subset(big_union[T](d))
        is_open_cover_iff[T](d, k)
        is_open_cover[T](d, k)
        is_open_cover[T](cover_adjoin_complement[T](s, c), k)
    }
}

/// The finite list obtained by discarding members outside the family `c`.
define cover_restrict_to_family[T](c: Set[T] -> Bool, items: List[Set[T]]) -> List[Set[T]] {
    items.filter(c)
}

lemma cover_restrict_to_family_cons_of_member[T](c: Set[T] -> Bool, head: Set[T], tail: List[Set[T]]) {
    c(head) implies cover_restrict_to_family[T](c, List.cons(head, tail)) =
        List.cons(head, cover_restrict_to_family[T](c, tail))
} by {
    if c(head) {
        List.cons(head, tail).filter(c) = List.cons(head, tail.filter(c))
    }
}

lemma cover_restrict_to_family_cons_of_not_member[T](c: Set[T] -> Bool, head: Set[T], tail: List[Set[T]]) {
    not c(head) implies cover_restrict_to_family[T](c, List.cons(head, tail)) =
        cover_restrict_to_family[T](c, tail)
} by {
    if not c(head) {
        List.cons(head, tail).filter(c) = tail.filter(c)
    }
}

lemma list_union_cover_restrict_cons_head[T](c: Set[T] -> Bool, head: Set[T], tail: List[Set[T]], x: T) {
    c(head) and head.contains(x) implies
        list_union_of_sets[T](cover_restrict_to_family[T](c, List.cons(head, tail))).contains(x)
} by {
    if c(head) and head.contains(x) {
        cover_restrict_to_family_cons_of_member[T](c, head, tail)
        list_union_of_sets_cons[T](head, cover_restrict_to_family[T](c, tail))
        union_contains_eq[T](head, list_union_of_sets[T](cover_restrict_to_family[T](c, tail)), x)
        list_union_of_sets[T](cover_restrict_to_family[T](c, List.cons(head, tail))).contains(x)
    }
}

lemma list_union_cover_restrict_cons_tail[T](c: Set[T] -> Bool, head: Set[T], tail: List[Set[T]], x: T) {
    list_union_of_sets[T](cover_restrict_to_family[T](c, tail)).contains(x) implies
        list_union_of_sets[T](cover_restrict_to_family[T](c, List.cons(head, tail))).contains(x)
} by {
    if list_union_of_sets[T](cover_restrict_to_family[T](c, tail)).contains(x) {
        if c(head) {
            cover_restrict_to_family_cons_of_member[T](c, head, tail)
            cover_restrict_to_family[T](c, List.cons(head, tail)) =
                List.cons(head, cover_restrict_to_family[T](c, tail))
            list_union_of_sets_cons[T](head, cover_restrict_to_family[T](c, tail))
            union_contains_eq[T](head, list_union_of_sets[T](cover_restrict_to_family[T](c, tail)), x)
            list_union_of_sets[T](cover_restrict_to_family[T](c, List.cons(head, tail))).contains(x)
        } else {
            cover_restrict_to_family_cons_of_not_member[T](c, head, tail)
            list_union_of_sets[T](cover_restrict_to_family[T](c, List.cons(head, tail))).contains(x)
        }
    }
}

/// Restricting a finite list to a family gives a finite subfamily of that family.
theorem cover_restrict_to_family_is_subfamily[T](c: Set[T] -> Bool, items: List[Set[T]]) {
    is_subfamily[T](c, cover_restrict_to_family[T](c, items))
} by {
    define p(xs: List[Set[T]]) -> Bool {
        is_subfamily[T](c, cover_restrict_to_family[T](c, xs))
    }
    cover_restrict_to_family[T](c, List.nil[Set[T]]) = List.nil[Set[T]]
    p(List.nil[Set[T]])
    forall(head: Set[T], tail: List[Set[T]]) {
        if p(tail) {
            if c(head) {
                cover_restrict_to_family_cons_of_member[T](c, head, tail)
                cover_restrict_to_family[T](c, List.cons(head, tail)) =
                    List.cons(head, cover_restrict_to_family[T](c, tail))
                is_subfamily_cons_mpr[T](c, head, cover_restrict_to_family[T](c, tail))
                is_subfamily[T](c, List.cons(head, cover_restrict_to_family[T](c, tail)))
                is_subfamily[T](c, cover_restrict_to_family[T](c, List.cons(head, tail)))
            } else {
                cover_restrict_to_family_cons_of_not_member[T](c, head, tail)
                is_subfamily[T](c, cover_restrict_to_family[T](c, List.cons(head, tail)))
            }
            p(List.cons(head, tail))
        }
        p(tail) implies p(List.cons(head, tail))
    }
    p(List.nil[Set[T]]) and forall(head: Set[T], tail: List[Set[T]]) {
        p(tail) implies p(List.cons(head, tail))
    }
    List.induction(function(xs: List[Set[T]]) { p(xs) })
    forall(xs: List[Set[T]]) { p(xs) }
}

/// Membership in a finite union comes from some member of the list.
lemma list_union_of_sets_contains_witness[T](items: List[Set[T]], x: T) {
    list_union_of_sets[T](items).contains(x) implies exists(member: Set[T]) {
        items.contains(member) and member.contains(x)
    }
} by {
    define p(xs: List[Set[T]]) -> Bool {
        list_union_of_sets[T](xs).contains(x) implies exists(member: Set[T]) {
            xs.contains(member) and member.contains(x)
        }
    }
    if list_union_of_sets[T](List.nil[Set[T]]).contains(x) {
        list_union_of_sets_nil[T]
        empty_set_contains_eq[T](x)
        false
    }
    p(List.nil[Set[T]])
    forall(head: Set[T], tail: List[Set[T]]) {
        if p(tail) {
            if not list_union_of_sets[T](List.cons(head, tail)).contains(x) {
                p(List.cons(head, tail))
            }
            if list_union_of_sets[T](List.cons(head, tail)).contains(x) {
                list_union_of_sets_cons[T](head, tail)
                union_contains_eq[T](head, list_union_of_sets[T](tail), x)
                if head.contains(x) {
                    exists(member: Set[T]) {
                        List.cons(head, tail).contains(member) and member.contains(x)
                    }
                } else {
                    p(tail) = (list_union_of_sets[T](tail).contains(x) implies exists(member: Set[T]) {
                        tail.contains(member) and member.contains(x)
                    })
                    let member: Set[T] satisfy {
                        tail.contains(member) and member.contains(x)
                    }
                    exists(result: Set[T]) {
                        List.cons(head, tail).contains(result) and result.contains(x)
                    }
                }
            }
            p(List.cons(head, tail))
        }
        p(tail) implies p(List.cons(head, tail))
    }
    p(List.nil[Set[T]]) and forall(head: Set[T], tail: List[Set[T]]) {
        p(tail) implies p(List.cons(head, tail))
    }
    List.induction(function(xs: List[Set[T]]) { p(xs) })
    forall(xs: List[Set[T]]) { p(xs) }
}

/// A member of a finite list whose set contains `x` contributes `x` to the finite union.
lemma list_union_of_sets_contains_of_member[T](items: List[Set[T]], member: Set[T], x: T) {
    items.contains(member) and member.contains(x) implies list_union_of_sets[T](items).contains(x)
} by {
    define p(xs: List[Set[T]]) -> Bool {
        xs.contains(member) and member.contains(x) implies list_union_of_sets[T](xs).contains(x)
    }
    if List.nil[Set[T]].contains(member) and member.contains(x) {
        false
    }
    p(List.nil[Set[T]])
    forall(head: Set[T], tail: List[Set[T]]) {
        if p(tail) {
            if List.cons(head, tail).contains(member) and member.contains(x) {
                list_union_of_sets_cons[T](head, tail)
                union_contains_eq[T](head, list_union_of_sets[T](tail), x)
                if head = member {
                    list_union_of_sets[T](List.cons(head, tail)).contains(x)
                } else {
                    tail.contains(member)
                    list_union_of_sets[T](List.cons(head, tail)).contains(x)
                }
            }
            p(List.cons(head, tail))
        }
        p(tail) implies p(List.cons(head, tail))
    }
    p(List.nil[Set[T]]) and forall(head: Set[T], tail: List[Set[T]]) {
        p(tail) implies p(List.cons(head, tail))
    }
    List.induction(function(xs: List[Set[T]]) { p(xs) })
    forall(xs: List[Set[T]]) { p(xs) }
}

/// Every member of a finite subfamily belongs to the original family.
lemma is_subfamily_contains[T](c: Set[T] -> Bool, items: List[Set[T]], member: Set[T]) {
    is_subfamily[T](c, items) and items.contains(member) implies c(member)
} by {
    define p(xs: List[Set[T]]) -> Bool {
        is_subfamily[T](c, xs) and xs.contains(member) implies c(member)
    }
    if is_subfamily[T](c, List.nil[Set[T]]) and List.nil[Set[T]].contains(member) {
        false
    }
    p(List.nil[Set[T]])
    forall(head: Set[T], tail: List[Set[T]]) {
        if p(tail) {
            if is_subfamily[T](c, List.cons(head, tail)) and List.cons(head, tail).contains(member) {
                is_subfamily_cons_mp[T](c, head, tail)
                if head = member {
                    c(member)
                } else {
                    c(member)
                }
            }
            p(List.cons(head, tail))
        }
        p(tail) implies p(List.cons(head, tail))
    }
    p(List.nil[Set[T]]) and forall(head: Set[T], tail: List[Set[T]]) {
        p(tail) implies p(List.cons(head, tail))
    }
    List.induction(function(xs: List[Set[T]]) { p(xs) })
    forall(xs: List[Set[T]]) { p(xs) }
}

/// A point of `s` covered by an augmented finite list is already covered after
/// discarding the adjoined complement of `s`.
theorem cover_restrict_to_family_covers_closed_member[T](
    s: Set[T],
    c: Set[T] -> Bool,
    items: List[Set[T]],
    x: T
) {
    is_subfamily[T](cover_adjoin_complement[T](s, c), items) and s.contains(x)
        and list_union_of_sets[T](items).contains(x)
        implies list_union_of_sets[T](cover_restrict_to_family[T](c, items)).contains(x)
} by {
    if is_subfamily[T](cover_adjoin_complement[T](s, c), items) and s.contains(x)
        and list_union_of_sets[T](items).contains(x) {
        list_union_of_sets_contains_witness[T](items, x)
        let member: Set[T] satisfy {
            items.contains(member) and member.contains(x)
        }
        is_subfamily_contains[T](cover_adjoin_complement[T](s, c), items, member)
        if c(member) {
            filter_contains_and[Set[T]](items, c, member)
            list_union_of_sets_contains_of_member[T](cover_restrict_to_family[T](c, items), member, x)
            list_union_of_sets[T](cover_restrict_to_family[T](c, items)).contains(x)
        } else {
            member = s.c
            compl_contains_eq[T](s, x)
            false
        }
        list_union_of_sets[T](cover_restrict_to_family[T](c, items)).contains(x)
    }
}

/// A finite subcover using an adjoined complement restricts to a finite subcover
/// of the closed set from the original family.
theorem cover_restrict_to_family_finite_subcover[T](
    s: Set[T],
    c: Set[T] -> Bool,
    items: List[Set[T]]
) {
    is_subfamily[T](cover_adjoin_complement[T](s, c), items)
        and s.subset(list_union_of_sets[T](items))
        implies exists(filtered: List[Set[T]]) {
            is_subfamily[T](c, filtered) and s.subset(list_union_of_sets[T](filtered))
        }
} by {
    if is_subfamily[T](cover_adjoin_complement[T](s, c), items)
        and s.subset(list_union_of_sets[T](items)) {
        let filtered: List[Set[T]] = cover_restrict_to_family[T](c, items)
        cover_restrict_to_family_is_subfamily[T](c, items)
        forall(x: T) {
            if s.contains(x) {
                subset_contains[T](s, list_union_of_sets[T](items), x)
                cover_restrict_to_family_covers_closed_member[T](s, c, items, x)
                list_union_of_sets[T](filtered).contains(x)
            }
        }
        subset_contains_eq[T](s, list_union_of_sets[T](filtered))
        s.subset(list_union_of_sets[T](filtered))
        exists(result: List[Set[T]]) {
            is_subfamily[T](c, result) and s.subset(list_union_of_sets[T](result))
        }
    }
}

/// A closed subset of a compact set is compact.
theorem closed_subset_of_compact_is_compact[T: TopologicalSpace](s: Set[T], k: Set[T]) {
    s.subset(k) and is_closed[T](s) and is_compact[T](k) implies is_compact[T](s)
} by {
    if s.subset(k) and is_closed[T](s) and is_compact[T](k) {
        forall(c: Set[T] -> Bool) {
            if is_open_cover[T](c, s) {
                let d: Set[T] -> Bool = cover_adjoin_complement[T](s, c)
                cover_adjoin_complement_is_open_cover_of_superset[T](s, k, c)
                compact_open_cover_has_finite_subcover[T](d, k)
                let items: List[Set[T]] satisfy {
                    is_subfamily[T](d, items) and k.subset(list_union_of_sets[T](items))
                }
                forall(x: T) {
                    if s.contains(x) {
                        subset_contains[T](s, k, x)
                        subset_contains[T](k, list_union_of_sets[T](items), x)
                        list_union_of_sets[T](items).contains(x)
                    }
                }
                subset_contains_eq[T](s, list_union_of_sets[T](items))
                s.subset(list_union_of_sets[T](items))
                cover_restrict_to_family_finite_subcover[T](s, c, items)
                exists(filtered: List[Set[T]]) {
                    is_subfamily[T](c, filtered) and s.subset(list_union_of_sets[T](filtered))
                }
            }
        }
    }
}

/// A closed subset of the union of two compact sets is compact.
theorem closed_subset_of_compact_union_is_compact[T: TopologicalSpace](
    s: Set[T],
    a: Set[T],
    b: Set[T]
) {
    is_closed[T](s) and s.subset(a.union(b)) and is_compact[T](a) and is_compact[T](b)
        implies is_compact[T](s)
} by {
    if is_closed[T](s) and s.subset(a.union(b)) and is_compact[T](a) and is_compact[T](b) {
        compact_union[T](a, b)
        closed_subset_of_compact_is_compact[T](s, a.union(b))
        is_compact[T](s)
    }
}

/// A closed subset of the continuous image of a compact set is compact.
theorem closed_subset_of_continuous_image_compact[X: TopologicalSpace, Y: TopologicalSpace](
    f: X -> Y,
    k: Set[X],
    s: Set[Y]
) {
    is_continuous[X, Y](f) and is_compact[X](k) and is_closed[Y](s)
        and s.subset(set_image[X, Y](k, f)) implies is_compact[Y](s)
} by {
    if is_continuous[X, Y](f) and is_compact[X](k) and is_closed[Y](s)
        and s.subset(set_image[X, Y](k, f)) {
        continuous_image_compact[X, Y](f, k)
        closed_subset_of_compact_is_compact[Y](s, set_image[X, Y](k, f))
        is_compact[Y](s)
    }
}
