from data.basic.set import Set
from analysis.topology.topological_space import TopologicalSpace, closure, interior, frontier,
    is_closed, open_eq_interior, closed_eq_closure
from analysis.topology.frontier_props import frontier_subset_closure

/// The frontier of an open set is its closure minus itself.
theorem open_frontier_eq_closure_diff_self[T: TopologicalSpace](s: Set[T]) {
    T.is_open(s) implies frontier(s) = closure(s).difference(s)
} by {
    if T.is_open(s) {
        open_eq_interior[T](s)
        closure(s).difference(interior(s)) = closure(s).difference(s)
        frontier(s) = closure(s).difference(s)
    }
}

/// The frontier of a closed set is the set minus its interior.
theorem closed_frontier_eq_self_diff_interior[T: TopologicalSpace](s: Set[T]) {
    is_closed[T](s) implies frontier(s) = s.difference(interior(s))
} by {
    if is_closed[T](s) {
        frontier(s) = closure(s).difference(interior(s))
        closed_eq_closure[T](s)
        s = closure(s)
        closure(s).difference(interior(s)) = s.difference(interior(s))
        frontier(s) = s.difference(interior(s))
    }
}

/// The frontier of a closed set is contained in the set.
theorem closed_frontier_subset_self[T: TopologicalSpace](s: Set[T]) {
    is_closed[T](s) implies frontier(s).subset(s)
} by {
    if is_closed[T](s) {
        frontier_subset_closure[T](s)
        frontier(s).subset(closure(s))
        closed_eq_closure[T](s)
        s = closure(s)
        closure(s) = s
        frontier(s).subset(s)
    }
}
