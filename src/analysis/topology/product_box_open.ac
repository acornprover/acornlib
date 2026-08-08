from data.basic.set import Set
from pair import Pair
from analysis.topology.topological_space import TopologicalSpace, generated_open, is_box,
    box, basis_member_is_open, is_open_in_product,
    is_open_in_product_iff

/// A box of open factors is a basic product box.
theorem box_is_box[X: TopologicalSpace, Y: TopologicalSpace](u: Set[X], v: Set[Y]) {
    X.is_open(u) and Y.is_open(v) implies is_box[X, Y](box(u, v))
} by {
    if X.is_open(u) and Y.is_open(v) {
        X.is_open(u) and Y.is_open(v) and box(u, v) = box(u, v)
        exists(uu: Set[X], vv: Set[Y]) {
            X.is_open(uu) and Y.is_open(vv) and box(u, v) = box(uu, vv)
        }
    }
}

/// A box of open factors generates an open set of the box basis.
theorem box_generated_open[X: TopologicalSpace, Y: TopologicalSpace](
    u: Set[X], v: Set[Y]) {
    X.is_open(u) and Y.is_open(v) implies generated_open(is_box[X, Y], box(u, v))
} by {
    if X.is_open(u) and Y.is_open(v) {
        box_is_box[X, Y](u, v)
        basis_member_is_open[Pair[X, Y]](is_box[X, Y], box(u, v))
    }
}

/// A box of open factors is open in the product topology.
theorem box_is_open_in_product[X: TopologicalSpace, Y: TopologicalSpace](
    u: Set[X], v: Set[Y]) {
    X.is_open(u) and Y.is_open(v) implies is_open_in_product[X, Y](box(u, v))
} by {
    if X.is_open(u) and Y.is_open(v) {
        box_generated_open[X, Y](u, v)
        generated_open(is_box[X, Y], box(u, v))
        is_open_in_product_iff[X, Y](box(u, v))
    }
}
