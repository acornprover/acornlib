from data.basic.set import Set, set_preimage, set_preimage_compl, compl_of_compl_is_self
from analysis.topology.topological_space import TopologicalSpace, is_closed, is_continuous

/// A function is continuous iff the preimage of every closed set is closed.
theorem continuous_iff_preimage_closed[X: TopologicalSpace, Y: TopologicalSpace](f: X -> Y) {
    is_continuous(f) = forall(c: Set[Y]) {
        is_closed(c) implies is_closed(set_preimage(f, c))
    }
} by {
    if is_continuous(f) {
        is_continuous(f) = forall(w: Set[Y]) {
            Y.is_open(w) implies X.is_open(set_preimage(f, w))
        }
        forall(c: Set[Y]) {
            if is_closed(c) {
                X.is_open(set_preimage(f, c.c))
                set_preimage_compl(f, c)
                is_closed(set_preimage(f, c))
            }
        }
    }
    if forall(c: Set[Y]) { is_closed(c) implies is_closed(set_preimage(f, c)) } {
        forall(v: Set[Y]) {
            if Y.is_open(v) {
                compl_of_compl_is_self(v)
                is_closed(v.c)
                set_preimage_compl(f, v)
                is_closed(set_preimage(f, v).c)
                compl_of_compl_is_self(set_preimage(f, v))
                X.is_open(set_preimage(f, v))
            }
        }
        is_continuous(f)
    }
}
