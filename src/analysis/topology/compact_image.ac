from list import List
from data.basic.set import Set, set_image, maps_into_set_image, set_image_contains_witness,
    set_preimage, set_preimage_contains_eq, empty_set_contains_eq,
    union_contains_eq, subset_contains, subset_contains_eq
from analysis.topology.topological_space import TopologicalSpace, big_union, big_union_contains_eq,
    big_union_contains_of_member, is_continuous, is_open_cover, is_open_cover_iff,
    is_compact, is_compact_iff,
    is_subfamily, is_subfamily_nil, is_subfamily_cons_mp,
    is_subfamily_cons_mpr, list_union_of_sets, list_union_of_sets_nil,
    list_union_of_sets_cons
from analysis.topology.continuous_preimage import continuous_open_preimage
from analysis.topology.compact_union import compact_open_cover_has_finite_subcover

/// Builds subset relation from pointwise membership implication.
theorem subset_of_forall_contains[K](a: Set[K], b: Set[K]) {
    (forall(x: K) { a.contains(x) implies b.contains(x) }) implies a.subset(b)
} by {
    if forall(x: K) { a.contains(x) implies b.contains(x) } {
        subset_contains_eq[K](a, b)
        a.subset(b)
    }
}

/// The open-set part of an open cover.
theorem open_cover_open_sets[T: TopologicalSpace](c: Set[T] -> Bool, s: Set[T]) {
    is_open_cover[T](c, s) implies forall(u: Set[T]) { c(u) implies T.is_open(u) }
} by {
    if is_open_cover[T](c, s) {
        is_open_cover_iff[T](c, s)
        forall(u: Set[T]) { c(u) implies T.is_open(u) }
    }
}

/// The containment part of an open cover.
theorem open_cover_subset_big_union[T: TopologicalSpace](c: Set[T] -> Bool, s: Set[T]) {
    is_open_cover[T](c, s) implies s.subset(big_union[T](c))
} by {
    if is_open_cover[T](c, s) {
        is_open_cover_iff[T](c, s)
        s.subset(big_union[T](c))
    }
}

/// Membership in the head set gives membership in the union of a cons list.
theorem list_union_of_sets_contains_head[K](head: Set[K], tail: List[Set[K]], x: K) {
    head.contains(x) implies list_union_of_sets[K](List.cons(head, tail)).contains(x)
} by {
    if head.contains(x) {
        list_union_of_sets_cons[K](head, tail)
        union_contains_eq[K](head, list_union_of_sets[K](tail), x)
        list_union_of_sets[K](List.cons(head, tail)).contains(x)
    }
}

/// Membership in the tail union gives membership in the union of a cons list.
theorem list_union_of_sets_contains_tail[K](head: Set[K], tail: List[Set[K]], x: K) {
    list_union_of_sets[K](tail).contains(x) implies
        list_union_of_sets[K](List.cons(head, tail)).contains(x)
} by {
    if list_union_of_sets[K](tail).contains(x) {
        list_union_of_sets_cons[K](head, tail)
        union_contains_eq[K](head, list_union_of_sets[K](tail), x)
        list_union_of_sets[K](List.cons(head, tail)).contains(x)
    }
}

/// The family of preimages of members of a cover family.
define preimage_cover_family_contains[X, Y](f: X -> Y, c: Set[Y] -> Bool, u: Set[X]) -> Bool {
    exists(v: Set[Y]) {
        c(v) and u = set_preimage(f, v)
    }
}

/// The cover family on the domain obtained by pulling a codomain family back along `f`.
define preimage_cover_family[X, Y](f: X -> Y, c: Set[Y] -> Bool) -> (Set[X] -> Bool) {
    preimage_cover_family_contains(f, c)
}

/// Membership in a pulled-back family is being the preimage of a member of the original family.
theorem preimage_cover_family_eq[X, Y](f: X -> Y, c: Set[Y] -> Bool, u: Set[X]) {
    preimage_cover_family(f, c)(u) = exists(v: Set[Y]) {
        c(v) and u = set_preimage(f, v)
    }
} by {
}

/// A chosen original cover member for a pulled-back cover member.
let preimage_cover_original[X, Y](f: X -> Y, c: Set[Y] -> Bool, u: Set[X]) -> result: Set[Y] satisfy {
    preimage_cover_family(f, c)(u) implies (c(result) and set_preimage(f, result) = u)
} by {
    if preimage_cover_family(f, c)(u) {
        preimage_cover_family_eq(f, c, u)
        let v: Set[Y] satisfy {
            c(v) and u = set_preimage(f, v)
        }
        exists(result: Set[Y]) {
            preimage_cover_family(f, c)(u) implies c(result) and set_preimage(f, result) = u
        }
    }
}

/// The chosen original member has the expected property whenever the pulled-back member belongs.
theorem preimage_cover_original_spec[X, Y](f: X -> Y, c: Set[Y] -> Bool, u: Set[X]) {
    preimage_cover_family(f, c)(u) implies (c(preimage_cover_original(f, c, u)) and
        set_preimage(f, preimage_cover_original(f, c, u)) = u)
} by {
    if preimage_cover_family(f, c)(u) {
        preimage_cover_original(f, c, u) = preimage_cover_original(f, c, u)
        set_preimage(f, preimage_cover_original(f, c, u)) = u
        c(preimage_cover_original(f, c, u)) and
            set_preimage(f, preimage_cover_original(f, c, u)) = u
    }
}

/// The finite list of original cover members corresponding to a finite list of pulled-back members.
define preimage_subcover_image_list[X, Y](f: X -> Y, c: Set[Y] -> Bool, items: List[Set[X]]) -> List[Set[Y]] {
    match items {
        List.nil {
            List.nil[Set[Y]]
        }
        List.cons(head, tail) {
            List.cons(preimage_cover_original(f, c, head), preimage_subcover_image_list(f, c, tail))
        }
    }
}

/// The pushed list for an empty pulled-back list is empty.
theorem preimage_subcover_image_list_nil[X, Y](f: X -> Y, c: Set[Y] -> Bool) {
    preimage_subcover_image_list(f, c, List.nil[Set[X]]) = List.nil[Set[Y]]
}

/// The pushed list of a cons is the corresponding cons of pushed original members.
theorem preimage_subcover_image_list_cons[X, Y](
    f: X -> Y,
    c: Set[Y] -> Bool,
    head: Set[X],
    tail: List[Set[X]]
) {
    preimage_subcover_image_list(f, c, List.cons(head, tail)) =
        List.cons(preimage_cover_original(f, c, head), preimage_subcover_image_list(f, c, tail))
}

/// A continuous map pulls an open cover of the image back to an open cover of the source set.
theorem preimage_cover_family_is_open_cover[X: TopologicalSpace, Y: TopologicalSpace](
    f: X -> Y,
    s: Set[X],
    c: Set[Y] -> Bool
) {
    is_continuous(f) and is_open_cover[Y](c, set_image(s, f))
        implies is_open_cover[X](preimage_cover_family(f, c), s)
} by {
    if is_continuous(f) and is_open_cover[Y](c, set_image(s, f)) {
        let d: Set[X] -> Bool = preimage_cover_family(f, c)
        open_cover_open_sets[Y](c, set_image(s, f))
        open_cover_subset_big_union[Y](c, set_image(s, f))
        forall(v: Set[Y]) { c(v) implies Y.is_open(v) }
        forall(u: Set[X]) {
            if d(u) {
                preimage_cover_family_eq(f, c, u)
                let v: Set[Y] satisfy {
                    c(v) and u = set_preimage(f, v)
                }
                continuous_open_preimage[X, Y](f, v)
                X.is_open(set_preimage(f, v))
                X.is_open(u)
            }
        }
        forall(x: X) {
            if s.contains(x) {
                maps_into_set_image(s, f, x)
                subset_contains[Y](set_image(s, f), big_union(c), f(x))
                big_union_contains_eq(c, f(x))
                let v: Set[Y] satisfy {
                    c(v) and v.contains(f(x))
                }
                set_preimage_contains_eq(f, v, x)
                preimage_cover_family_eq(f, c, set_preimage(f, v))
                d(set_preimage(f, v))
                big_union_contains_of_member(d, set_preimage(f, v), x)
                big_union(d).contains(x)
            }
            s.contains(x) implies big_union(d).contains(x)
        }
        subset_of_forall_contains[X](s, big_union(d))
        s.subset(big_union(d))
        is_open_cover_iff[X](d, s)
        is_open_cover[X](d, s)
        is_open_cover[X](preimage_cover_family(f, c), s)
    }
}

/// Pushing a pulled-back finite subfamily produces a finite subfamily of the original cover.
theorem preimage_subcover_image_list_is_subfamily[X, Y](
    f: X -> Y,
    c: Set[Y] -> Bool,
    items: List[Set[X]]
) {
    is_subfamily[X](preimage_cover_family(f, c), items) implies
        is_subfamily[Y](c, preimage_subcover_image_list(f, c, items))
} by {
    let d: Set[X] -> Bool = preimage_cover_family(f, c)
    define p(xs: List[Set[X]]) -> Bool {
        is_subfamily[X](d, xs) implies is_subfamily[Y](c, preimage_subcover_image_list(f, c, xs))
    }
    if is_subfamily[X](d, List.nil[Set[X]]) {
        preimage_subcover_image_list_nil[X, Y](f, c)
        is_subfamily_nil[Y](c)
        is_subfamily[Y](c, preimage_subcover_image_list(f, c, List.nil[Set[X]]))
    }
    p(List.nil[Set[X]])
    forall(head: Set[X], tail: List[Set[X]]) {
        if p(tail) {
            if is_subfamily[X](d, List.cons(head, tail)) {
                is_subfamily_cons_mp[X](d, head, tail)
                is_subfamily[Y](c, preimage_subcover_image_list(f, c, tail))
                preimage_cover_original_spec[X, Y](f, c, head)
                c(preimage_cover_original(f, c, head))
                preimage_subcover_image_list_cons(f, c, head, tail)
                is_subfamily_cons_mpr[Y](c, preimage_cover_original(f, c, head),
                    preimage_subcover_image_list(f, c, tail))
                is_subfamily[Y](c, preimage_subcover_image_list(f, c, List.cons(head, tail)))
            }
            p(List.cons(head, tail))
        }
        p(tail) implies p(List.cons(head, tail))
    }
    p(List.nil[Set[X]]) and forall(head: Set[X], tail: List[Set[X]]) {
        p(tail) implies p(List.cons(head, tail))
    }
    List.induction(function(xs: List[Set[X]]) { p(xs) })
    forall(xs: List[Set[X]]) { p(xs) }
}

/// If a point is in the union of a pulled-back finite subfamily, its image is in the
/// union of the corresponding original finite subfamily.
theorem preimage_subcover_image_list_covers_image[X, Y](
    f: X -> Y,
    c: Set[Y] -> Bool,
    items: List[Set[X]],
    x: X
) {
    is_subfamily[X](preimage_cover_family(f, c), items) and
        list_union_of_sets[X](items).contains(x)
        implies list_union_of_sets[Y](preimage_subcover_image_list(f, c, items)).contains(f(x))
} by {
    let d: Set[X] -> Bool = preimage_cover_family(f, c)
    define p(xs: List[Set[X]]) -> Bool {
        is_subfamily[X](d, xs) and list_union_of_sets[X](xs).contains(x)
            implies list_union_of_sets[Y](preimage_subcover_image_list(f, c, xs)).contains(f(x))
    }
    if is_subfamily[X](d, List.nil[Set[X]]) and list_union_of_sets[X](List.nil[Set[X]]).contains(x) {
        list_union_of_sets_nil[X]
        empty_set_contains_eq[X](x)
        false
    }
    p(List.nil[Set[X]])
    forall(head: Set[X], tail: List[Set[X]]) {
        if p(tail) {
            if is_subfamily[X](d, List.cons(head, tail)) and
                list_union_of_sets[X](List.cons(head, tail)).contains(x) {
                is_subfamily_cons_mp[X](d, head, tail)
                preimage_cover_original_spec[X, Y](f, c, head)
                set_preimage(f, preimage_cover_original(f, c, head)) = head
                list_union_of_sets_cons[X](head, tail)
                union_contains_eq(head, list_union_of_sets[X](tail), x)
                if head.contains(x) {
                    set_preimage_contains_eq(f, preimage_cover_original(f, c, head), x)
                    preimage_cover_original(f, c, head).contains(f(x))
                    preimage_subcover_image_list_cons(f, c, head, tail)
                    list_union_of_sets_contains_head[Y](preimage_cover_original(f, c, head),
                        preimage_subcover_image_list(f, c, tail), f(x))
                    list_union_of_sets[Y](preimage_subcover_image_list(f, c,
                        List.cons(head, tail))).contains(f(x))
                } else {
                    p(tail) = (is_subfamily[X](d, tail) and list_union_of_sets[X](tail).contains(x)
                        implies list_union_of_sets[Y](preimage_subcover_image_list(f, c, tail)).contains(f(x)))
                    list_union_of_sets[Y](preimage_subcover_image_list(f, c, tail)).contains(f(x))
                    preimage_subcover_image_list_cons(f, c, head, tail)
                    list_union_of_sets_contains_tail[Y](preimage_cover_original(f, c, head),
                        preimage_subcover_image_list(f, c, tail), f(x))
                    list_union_of_sets[Y](preimage_subcover_image_list(f, c,
                        List.cons(head, tail))).contains(f(x))
                }
            }
            p(List.cons(head, tail))
        }
        p(tail) implies p(List.cons(head, tail))
    }
    p(List.nil[Set[X]]) and forall(head: Set[X], tail: List[Set[X]]) {
        p(tail) implies p(List.cons(head, tail))
    }
    List.induction(function(xs: List[Set[X]]) { p(xs) })
    forall(xs: List[Set[X]]) { p(xs) }
    if is_subfamily[X](preimage_cover_family(f, c), items) and list_union_of_sets[X](items).contains(x) {
        list_union_of_sets[Y](preimage_subcover_image_list(f, c, items)).contains(f(x))
    }
}

/// An open cover of the image of a compact set under a continuous map has a finite subcover.
theorem continuous_image_open_cover_has_finite_subcover[X: TopologicalSpace, Y: TopologicalSpace](
    f: X -> Y,
    s: Set[X],
    c: Set[Y] -> Bool
) {
    is_continuous(f) and is_compact[X](s) and is_open_cover[Y](c, set_image(s, f))
        implies exists(items: List[Set[Y]]) {
            is_subfamily[Y](c, items) and set_image(s, f).subset(list_union_of_sets[Y](items))
        }
} by {
    if is_continuous(f) and is_compact[X](s) and is_open_cover[Y](c, set_image(s, f)) {
        let d: Set[X] -> Bool = preimage_cover_family(f, c)
        preimage_cover_family_is_open_cover[X, Y](f, s, c)
        compact_open_cover_has_finite_subcover[X](d, s)
        let items_x: List[Set[X]] satisfy {
            is_subfamily[X](d, items_x) and s.subset(list_union_of_sets[X](items_x))
        }
        let items_y: List[Set[Y]] = preimage_subcover_image_list(f, c, items_x)
        preimage_subcover_image_list_is_subfamily[X, Y](f, c, items_x)
        is_subfamily[Y](c, items_y)
        forall(y: Y) {
            if set_image(s, f).contains(y) {
                set_image_contains_witness(s, f, y)
                let x: X satisfy {
                    s.contains(x) and y = f(x)
                }
                subset_contains[X](s, list_union_of_sets[X](items_x), x)
                preimage_subcover_image_list_covers_image[X, Y](f, c, items_x, x)
                list_union_of_sets[Y](items_y).contains(f(x))
                list_union_of_sets[Y](items_y).contains(y)
            }
        }
        subset_of_forall_contains[Y](set_image(s, f), list_union_of_sets[Y](items_y))
        exists(items: List[Set[Y]]) {
            is_subfamily[Y](c, items) and set_image(s, f).subset(list_union_of_sets[Y](items))
        }
    }
}

/// The image of a compact set under a continuous map is compact.
theorem continuous_image_compact[X: TopologicalSpace, Y: TopologicalSpace](
    f: X -> Y,
    s: Set[X]
) {
    is_continuous(f) and is_compact[X](s) implies is_compact[Y](set_image(s, f))
} by {
    if is_continuous(f) and is_compact[X](s) {
        is_compact_iff[Y](set_image(s, f))
        if not is_compact[Y](set_image(s, f)) {
            let c: Set[Y] -> Bool satisfy {
                is_open_cover[Y](c, set_image(s, f)) and not exists(items: List[Set[Y]]) {
                    is_subfamily[Y](c, items) and set_image(s, f).subset(list_union_of_sets[Y](items))
                }
            }
            continuous_image_open_cover_has_finite_subcover[X, Y](f, s, c)
            false
        }
    }
}
