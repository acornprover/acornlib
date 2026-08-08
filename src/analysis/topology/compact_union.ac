from list import List
from data.basic.set import Set, subset_trans, sets_subset_union, sets_subset_contain_union,
    union_with_empty_is_self, union_comm, union_assoc, insert_eq_union_singleton
from analysis.topology.topological_space import TopologicalSpace, big_union, is_open_cover,
    is_open_cover_iff, is_compact, is_compact_iff, is_subfamily,
    is_subfamily_cons_mp, is_subfamily_cons_mpr, list_union_of_sets,
    list_union_of_sets_nil, list_union_of_sets_cons, singleton_is_compact

/// Appending finite lists of sets unions their covered sets.
theorem list_union_of_sets_append[T](left: List[Set[T]], right: List[Set[T]]) {
    list_union_of_sets[T](left + right) =
        list_union_of_sets[T](left).union(list_union_of_sets[T](right))
} by {
    define p(items: List[Set[T]]) -> Bool {
        list_union_of_sets[T](items + right) =
            list_union_of_sets[T](items).union(list_union_of_sets[T](right))
    }
    list_union_of_sets_nil[T]
    union_comm(Set[T].empty_set, list_union_of_sets[T](right))
    union_with_empty_is_self(list_union_of_sets[T](right))
    p(List.nil[Set[T]])
    forall(head: Set[T], tail: List[Set[T]]) {
        if p(tail) {
            list_union_of_sets_cons(head, tail + right)
            list_union_of_sets_cons(head, tail)
            list_union_of_sets[T](tail + right) =
                list_union_of_sets[T](tail).union(list_union_of_sets[T](right))
            union_assoc(head, list_union_of_sets[T](tail), list_union_of_sets[T](right))
            p(List.cons(head, tail))
        }
    }
    forall(head: Set[T], tail: List[Set[T]]) {
        p(tail) implies p(List.cons(head, tail))
    }
    p(List.nil[Set[T]]) and forall(head: Set[T], tail: List[Set[T]]) {
        p(tail) implies p(List.cons(head, tail))
    }
    List.induction(function(items: List[Set[T]]) { p(items) })
    forall(items: List[Set[T]]) { p(items) }
}

/// Appending two finite subfamilies remains a finite subfamily.
theorem is_subfamily_append[T](c: Set[T] -> Bool, left: List[Set[T]], right: List[Set[T]]) {
    is_subfamily[T](c, left) and is_subfamily[T](c, right)
        implies is_subfamily[T](c, left + right)
} by {
    define p(items: List[Set[T]]) -> Bool {
        is_subfamily[T](c, items) and is_subfamily[T](c, right)
            implies is_subfamily[T](c, items + right)
    }
    if is_subfamily[T](c, List.nil[Set[T]]) and is_subfamily[T](c, right) {
        is_subfamily[T](c, List.nil[Set[T]] + right)
    }
    p(List.nil[Set[T]])
    forall(head: Set[T], tail: List[Set[T]]) {
        if p(tail) {
            if is_subfamily[T](c, List.cons(head, tail)) and is_subfamily[T](c, right) {
                is_subfamily_cons_mp(c, head, tail)
                is_subfamily_cons_mpr(c, head, tail + right)
                is_subfamily[T](c, List.cons(head, tail + right))
                p(List.cons(head, tail))
            }
            if not (is_subfamily[T](c, List.cons(head, tail)) and is_subfamily[T](c, right)) {
            }
            p(List.cons(head, tail))
        }
    }
    forall(head: Set[T], tail: List[Set[T]]) {
        p(tail) implies p(List.cons(head, tail))
    }
    p(List.nil[Set[T]]) and forall(head: Set[T], tail: List[Set[T]]) {
        p(tail) implies p(List.cons(head, tail))
    }
    List.induction(function(items: List[Set[T]]) { p(items) })
    forall(items: List[Set[T]]) { p(items) }
    if is_subfamily[T](c, left) and is_subfamily[T](c, right) {
        is_subfamily[T](c, left + right)
    }
}

/// Combining two finite subcovers by appending their finite subfamilies covers
/// the binary union.
theorem union_subcover_append[T](s: Set[T], t: Set[T], left: List[Set[T]], right: List[Set[T]]) {
    s.subset(list_union_of_sets[T](left)) and t.subset(list_union_of_sets[T](right))
        implies s.union(t).subset(list_union_of_sets[T](left + right))
} by {
    if s.subset(list_union_of_sets[T](left)) and t.subset(list_union_of_sets[T](right)) {
        list_union_of_sets_append(left, right)
        let combined: Set[T] = list_union_of_sets[T](left).union(list_union_of_sets[T](right))
        sets_subset_union(list_union_of_sets[T](left), list_union_of_sets[T](right))
        subset_trans(s, list_union_of_sets[T](left), combined)
        subset_trans(t, list_union_of_sets[T](right), combined)
        t.subset(combined)
        sets_subset_contain_union(s, t, combined)
        s.union(t).subset(combined)
        s.union(t).subset(list_union_of_sets[T](left + right))
    }
}

/// A compact set admits a finite subcover from any open cover.
lemma compact_open_cover_finite_subcover[T: TopologicalSpace](s: Set[T], c: Set[T] -> Bool) {
    is_compact[T](s) and is_open_cover[T](c, s) implies exists(items: List[Set[T]]) {
        is_subfamily[T](c, items) and s.subset(list_union_of_sets[T](items))
    }
} by {
    if is_compact[T](s) and is_open_cover[T](c, s) {
        is_compact_iff[T](s)
        if not exists(items: List[Set[T]]) {
            is_subfamily[T](c, items) and s.subset(list_union_of_sets[T](items))
        } {
            false
        }
    }
}

/// A compact set admits a finite subcover for each open cover.
theorem compact_open_cover_has_finite_subcover[T: TopologicalSpace](c: Set[T] -> Bool, s: Set[T]) {
    is_compact[T](s) and is_open_cover[T](c, s) implies exists(items: List[Set[T]]) {
        is_subfamily[T](c, items) and s.subset(list_union_of_sets[T](items))
    }
} by {
    if is_compact[T](s) and is_open_cover[T](c, s) {
        compact_open_cover_finite_subcover(s, c)
    }
}

/// The union of two compact subsets is compact.
theorem compact_union[T: TopologicalSpace](s: Set[T], t: Set[T]) {
    is_compact[T](s) and is_compact[T](t) implies is_compact[T](s.union(t))
} by {
    if is_compact[T](s) and is_compact[T](t) {
        forall(c: Set[T] -> Bool) {
            if is_open_cover[T](c, s.union(t)) {
                is_open_cover_iff[T](c, s.union(t))
                forall(u: Set[T]) { c(u) implies T.is_open(u) }

                sets_subset_union(s, t)
                subset_trans(s, s.union(t), big_union[T](c))
                is_open_cover_iff[T](c, s)

                subset_trans(t, s.union(t), big_union[T](c))
                is_open_cover_iff[T](c, t)

                compact_open_cover_finite_subcover(s, c)
                let left: List[Set[T]] satisfy {
                    is_subfamily[T](c, left) and s.subset(list_union_of_sets[T](left))
                }

                compact_open_cover_finite_subcover(t, c)
                let right: List[Set[T]] satisfy {
                    is_subfamily[T](c, right) and t.subset(list_union_of_sets[T](right))
                }

                is_subfamily_append(c, left, right)
                union_subcover_append(s, t, left, right)
                exists(items: List[Set[T]]) {
                    is_subfamily[T](c, items) and s.union(t).subset(list_union_of_sets[T](items))
                }
            }
        }
    }
}

/// Inserting one point into a compact set preserves compactness.
theorem compact_insert[T: TopologicalSpace](s: Set[T], x: T) {
    is_compact[T](s) implies is_compact[T](s.insert(x))
} by {
    if is_compact[T](s) {
        singleton_is_compact[T](x)
        compact_union(s, Set[T].singleton(x))
        insert_eq_union_singleton(s, x)
        is_compact[T](s.insert(x))
    }
}
