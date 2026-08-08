from data.basic.set import Set, set_preimage, set_preimage_contains_eq
from data.basic.functions import injective_fn_ne
from analysis.topology.topological_space import TopologicalSpace, is_homeomorphism,
    is_t0_space, is_t0_space_iff, is_t1_space, is_t1_space_iff,
    is_hausdorff, is_hausdorff_iff, disjoint_not_both
from analysis.topology.homeomorphism_inverse import homeomorphism_inverse
from analysis.topology.homeomorphism_set_transport import homeomorphism_forward_injective,
    homeomorphism_preimage_open

/// A homeomorphism transports the T0 separation property forward.
theorem homeomorphism_preserves_t0[X: TopologicalSpace, Y: TopologicalSpace](
    f: X -> Y, g: Y -> X) {
    is_homeomorphism[X, Y](f, g) and is_t0_space[X](X.is_open)
        implies is_t0_space[Y](Y.is_open)
} by {
    if is_homeomorphism[X, Y](f, g) and is_t0_space[X](X.is_open) {
        homeomorphism_inverse[X, Y](f, g)
        is_homeomorphism[Y, X](g, f)
        homeomorphism_forward_injective[Y, X](g, f)

        is_t0_space_iff[Y](Y.is_open)
        forall(y1: Y, y2: Y) {
            if y1 != y2 {
                injective_fn_ne[Y, X](g, y1, y2)
                g(y1) != g(y2)
                is_t0_space_iff[X](X.is_open)
                let u: Set[X] satisfy {
                    X.is_open(u) and ((u.contains(g(y1)) and not u.contains(g(y2)))
                        or (u.contains(g(y2)) and not u.contains(g(y1))))
                }
                let pre = set_preimage(g, u)
                homeomorphism_preimage_open[Y, X](g, f, u)
                Y.is_open(pre)
                if u.contains(g(y1)) and not u.contains(g(y2)) {
                    set_preimage_contains_eq(g, u, y1)
                    pre.contains(y1)
                    set_preimage_contains_eq(g, u, y2)
                    not pre.contains(y2)
                    Y.is_open(pre) and ((pre.contains(y1) and not pre.contains(y2))
                        or (pre.contains(y2) and not pre.contains(y1)))
                    exists(w: Set[Y]) {
                        Y.is_open(w) and ((w.contains(y1) and not w.contains(y2))
                            or (w.contains(y2) and not w.contains(y1)))
                    }
                } else {
                    u.contains(g(y2)) and not u.contains(g(y1))
                    set_preimage_contains_eq(g, u, y2)
                    pre.contains(y2)
                    set_preimage_contains_eq(g, u, y1)
                    not pre.contains(y1)
                    Y.is_open(pre) and ((pre.contains(y1) and not pre.contains(y2))
                        or (pre.contains(y2) and not pre.contains(y1)))
                    exists(w: Set[Y]) {
                        Y.is_open(w) and ((w.contains(y1) and not w.contains(y2))
                            or (w.contains(y2) and not w.contains(y1)))
                    }
                }
            }
        }
    }
}

/// A homeomorphism transports the T0 separation property backward.
theorem homeomorphism_reflects_t0[X: TopologicalSpace, Y: TopologicalSpace](
    f: X -> Y, g: Y -> X) {
    is_homeomorphism[X, Y](f, g) and is_t0_space[Y](Y.is_open)
        implies is_t0_space[X](X.is_open)
} by {
    if is_homeomorphism[X, Y](f, g) and is_t0_space[Y](Y.is_open) {
        homeomorphism_inverse[X, Y](f, g)
        is_homeomorphism[Y, X](g, f)
        homeomorphism_preserves_t0[Y, X](g, f)
        is_t0_space[X](X.is_open)
    }
}

/// Homeomorphic spaces satisfy the T0 separation property simultaneously.
theorem homeomorphism_t0_iff[X: TopologicalSpace, Y: TopologicalSpace](
    f: X -> Y, g: Y -> X) {
    is_homeomorphism[X, Y](f, g) implies
        (is_t0_space[X](X.is_open) = is_t0_space[Y](Y.is_open))
} by {
    if is_homeomorphism[X, Y](f, g) {
        if is_t0_space[X](X.is_open) {
            homeomorphism_preserves_t0[X, Y](f, g)
            is_t0_space[Y](Y.is_open)
        }
        if is_t0_space[Y](Y.is_open) {
            homeomorphism_reflects_t0[X, Y](f, g)
            is_t0_space[X](X.is_open)
        }
        is_t0_space[X](X.is_open) = is_t0_space[Y](Y.is_open)
    }
}

/// A homeomorphism transports the T1 separation property forward.
theorem homeomorphism_preserves_t1[X: TopologicalSpace, Y: TopologicalSpace](
    f: X -> Y, g: Y -> X) {
    is_homeomorphism[X, Y](f, g) and is_t1_space[X](X.is_open)
        implies is_t1_space[Y](Y.is_open)
} by {
    if is_homeomorphism[X, Y](f, g) and is_t1_space[X](X.is_open) {
        homeomorphism_inverse[X, Y](f, g)
        is_homeomorphism[Y, X](g, f)
        homeomorphism_forward_injective[Y, X](g, f)

        is_t1_space_iff[Y](Y.is_open)
        forall(y1: Y, y2: Y) {
            if y1 != y2 {
                injective_fn_ne[Y, X](g, y1, y2)
                g(y1) != g(y2)
                is_t1_space_iff[X](X.is_open)
                let u: Set[X] satisfy {
                    X.is_open(u) and u.contains(g(y1)) and not u.contains(g(y2))
                }
                let pre = set_preimage(g, u)
                homeomorphism_preimage_open[Y, X](g, f, u)
                Y.is_open(pre)
                set_preimage_contains_eq(g, u, y1)
                pre.contains(y1)
                set_preimage_contains_eq(g, u, y2)
                not pre.contains(y2)
                Y.is_open(pre) and pre.contains(y1) and not pre.contains(y2)
                exists(w: Set[Y]) {
                    Y.is_open(w) and w.contains(y1) and not w.contains(y2)
                }
            }
        }
    }
}

/// A homeomorphism transports the T1 separation property backward.
theorem homeomorphism_reflects_t1[X: TopologicalSpace, Y: TopologicalSpace](
    f: X -> Y, g: Y -> X) {
    is_homeomorphism[X, Y](f, g) and is_t1_space[Y](Y.is_open)
        implies is_t1_space[X](X.is_open)
} by {
    if is_homeomorphism[X, Y](f, g) and is_t1_space[Y](Y.is_open) {
        homeomorphism_inverse[X, Y](f, g)
        is_homeomorphism[Y, X](g, f)
        homeomorphism_preserves_t1[Y, X](g, f)
        is_t1_space[X](X.is_open)
    }
}

/// Homeomorphic spaces satisfy the T1 separation property simultaneously.
theorem homeomorphism_t1_iff[X: TopologicalSpace, Y: TopologicalSpace](
    f: X -> Y, g: Y -> X) {
    is_homeomorphism[X, Y](f, g) implies
        (is_t1_space[X](X.is_open) = is_t1_space[Y](Y.is_open))
} by {
    if is_homeomorphism[X, Y](f, g) {
        if is_t1_space[X](X.is_open) {
            homeomorphism_preserves_t1[X, Y](f, g)
            is_t1_space[Y](Y.is_open)
        }
        if is_t1_space[Y](Y.is_open) {
            homeomorphism_reflects_t1[X, Y](f, g)
            is_t1_space[X](X.is_open)
        }
        is_t1_space[X](X.is_open) = is_t1_space[Y](Y.is_open)
    }
}

/// A homeomorphism transports the Hausdorff separation property forward.
theorem homeomorphism_preserves_hausdorff[X: TopologicalSpace, Y: TopologicalSpace](
    f: X -> Y, g: Y -> X) {
    is_homeomorphism[X, Y](f, g) and is_hausdorff[X](X.is_open)
        implies is_hausdorff[Y](Y.is_open)
} by {
    if is_homeomorphism[X, Y](f, g) and is_hausdorff[X](X.is_open) {
        homeomorphism_inverse[X, Y](f, g)
        is_homeomorphism[Y, X](g, f)
        homeomorphism_forward_injective[Y, X](g, f)

        is_hausdorff_iff[Y](Y.is_open)
        forall(y1: Y, y2: Y) {
            if y1 != y2 {
                injective_fn_ne[Y, X](g, y1, y2)
                g(y1) != g(y2)
                is_hausdorff_iff[X](X.is_open)
                let u: Set[X] satisfy {
                    exists(vv: Set[X]) {
                        X.is_open(u) and X.is_open(vv) and u.contains(g(y1))
                            and vv.contains(g(y2)) and u.is_disjoint(vv)
                    }
                }
                let v: Set[X] satisfy {
                    X.is_open(u) and X.is_open(v) and u.contains(g(y1))
                        and v.contains(g(y2)) and u.is_disjoint(v)
                }
                let pre_u = set_preimage(g, u)
                let pre_v = set_preimage(g, v)
                homeomorphism_preimage_open[Y, X](g, f, u)
                homeomorphism_preimage_open[Y, X](g, f, v)
                Y.is_open(pre_u)
                Y.is_open(pre_v)
                set_preimage_contains_eq(g, u, y1)
                pre_u.contains(y1)
                set_preimage_contains_eq(g, v, y2)
                pre_v.contains(y2)
                forall(z: Y) {
                    if pre_u.contains(z) and pre_v.contains(z) {
                        set_preimage_contains_eq(g, u, z)
                        u.contains(g(z))
                        set_preimage_contains_eq(g, v, z)
                        v.contains(g(z))
                        disjoint_not_both[X](u, v, g(z))
                        not (u.contains(g(z)) and v.contains(g(z)))
                        false
                    }
                }
                pre_u.is_disjoint(pre_v)
                (Y.is_open(pre_u) and Y.is_open(pre_v) and pre_u.contains(y1)
                    and pre_v.contains(y2) and pre_u.is_disjoint(pre_v))
                exists(a: Set[Y], b: Set[Y]) {
                    (Y.is_open(a) and Y.is_open(b) and a.contains(y1) and b.contains(y2)
                        and a.is_disjoint(b))
                }
            }
        }
    }
}

/// A homeomorphism transports the Hausdorff separation property backward.
theorem homeomorphism_reflects_hausdorff[X: TopologicalSpace, Y: TopologicalSpace](
    f: X -> Y, g: Y -> X) {
    is_homeomorphism[X, Y](f, g) and is_hausdorff[Y](Y.is_open)
        implies is_hausdorff[X](X.is_open)
} by {
    if is_homeomorphism[X, Y](f, g) and is_hausdorff[Y](Y.is_open) {
        homeomorphism_inverse[X, Y](f, g)
        is_homeomorphism[Y, X](g, f)
        homeomorphism_preserves_hausdorff[Y, X](g, f)
        is_hausdorff[X](X.is_open)
    }
}

/// Homeomorphic spaces satisfy the Hausdorff separation property simultaneously.
theorem homeomorphism_hausdorff_iff[X: TopologicalSpace, Y: TopologicalSpace](
    f: X -> Y, g: Y -> X) {
    is_homeomorphism[X, Y](f, g) implies
        (is_hausdorff[X](X.is_open) = is_hausdorff[Y](Y.is_open))
} by {
    if is_homeomorphism[X, Y](f, g) {
        if is_hausdorff[X](X.is_open) {
            homeomorphism_preserves_hausdorff[X, Y](f, g)
            is_hausdorff[Y](Y.is_open)
        }
        if is_hausdorff[Y](Y.is_open) {
            homeomorphism_reflects_hausdorff[X, Y](f, g)
            is_hausdorff[X](X.is_open)
        }
        is_hausdorff[X](X.is_open) = is_hausdorff[Y](Y.is_open)
    }
}
