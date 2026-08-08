/// Pointwise composition theorem for continuity at a point.

from data.basic.set import Set, set_preimage, set_preimage_compose
from data.basic.functions import compose
from analysis.topology.topological_space import TopologicalSpace, is_neighborhood, is_continuous_at,
    is_continuous, continuous_imp_continuous_at

/// A continuous-at map sends neighborhoods of the image point back to neighborhoods of the source point.
theorem continuous_at_preimage_neighborhood[X: TopologicalSpace, Y: TopologicalSpace](
    f: X -> Y, x: X, v: Set[Y]
) {
    is_continuous_at[X, Y](f, x) and is_neighborhood(v, f(x))
        implies is_neighborhood(set_preimage(f, v), x)
} by {
    if is_continuous_at[X, Y](f, x) and is_neighborhood(v, f(x)) {
        if not is_neighborhood(set_preimage(f, v), x) {
            function(w: Set[Y]) {
                is_neighborhood(w, f(x)) implies is_neighborhood(set_preimage(f, w), x)
            }(v)
            false
        }
        is_neighborhood(set_preimage(f, v), x)
    }
}

/// A neighborhood of the composed value pulls back to a neighborhood under the composed map.
theorem continuous_at_compose_preimage_neighborhood[X: TopologicalSpace, Y: TopologicalSpace, Z: TopologicalSpace](
    f: X -> Y, g: Y -> Z, x: X, w: Set[Z]
) {
    is_continuous_at[X, Y](f, x) and is_continuous_at[Y, Z](g, f(x))
    and is_neighborhood(w, compose(g, f, x))
    implies is_neighborhood(set_preimage(compose(g, f), w), x)
} by {
    if is_continuous_at[X, Y](f, x) and is_continuous_at[Y, Z](g, f(x))
    and is_neighborhood(w, compose(g, f, x)) {
        continuous_at_preimage_neighborhood[Y, Z](g, f(x), w)
        continuous_at_preimage_neighborhood[X, Y](f, x, set_preimage(g, w))
        is_neighborhood(set_preimage(f, set_preimage(g, w)), x)
        set_preimage_compose(g, f, w)
        is_neighborhood(set_preimage(compose(g, f), w), x)
    }
}

/// If `f` is continuous at `x` and `g` is continuous at `f(x)`, then `g ∘ f` is continuous at `x`.
theorem continuous_at_compose[X: TopologicalSpace, Y: TopologicalSpace, Z: TopologicalSpace](
    f: X -> Y, g: Y -> Z, x: X
) {
    is_continuous_at[X, Y](f, x) and is_continuous_at[Y, Z](g, f(x))
        implies is_continuous_at[X, Z](compose(g, f), x)
} by {
    if is_continuous_at[X, Y](f, x) and is_continuous_at[Y, Z](g, f(x)) {
        forall(w: Set[Z]) {
            if is_neighborhood(w, compose(g, f, x)) {
                continuous_at_compose_preimage_neighborhood(f, g, x, w)
                is_neighborhood(set_preimage(compose(g, f), w), x)
            }
        }
        let c = compose(g, f)
        let p: Set[Z] -> Bool = function(w: Set[Z]) {
            is_neighborhood(w, c(x)) implies is_neighborhood(set_preimage(c, w), x)
        }
        forall(w: Set[Z]) {
            if is_neighborhood(w, c(x)) {
                is_neighborhood(w, compose(g, f, x))
                continuous_at_compose_preimage_neighborhood(f, g, x, w)
                p(w)
            }
        }
        forall(w: Set[Z]) {
            p(w)
        }
        is_continuous_at[X, Z](compose(g, f), x)
    }
}

/// If `f` is globally continuous and `g` is continuous at `f(x)`, then `g ∘ f` is continuous at `x`.
theorem continuous_at_compose_left_global[X: TopologicalSpace, Y: TopologicalSpace, Z: TopologicalSpace](
    f: X -> Y, g: Y -> Z, x: X
) {
    is_continuous[X, Y](f) and is_continuous_at[Y, Z](g, f(x))
        implies is_continuous_at[X, Z](compose(g, f), x)
} by {
    if is_continuous[X, Y](f) and is_continuous_at[Y, Z](g, f(x)) {
        continuous_imp_continuous_at[X, Y](f, x)
        is_continuous_at[X, Y](f, x)
        continuous_at_compose[X, Y, Z](f, g, x)
        is_continuous_at[X, Z](compose(g, f), x)
    }
}

/// If `f` is continuous at `x` and `g` is globally continuous, then `g ∘ f` is continuous at `x`.
theorem continuous_at_compose_right_global[X: TopologicalSpace, Y: TopologicalSpace, Z: TopologicalSpace](
    f: X -> Y, g: Y -> Z, x: X
) {
    is_continuous_at[X, Y](f, x) and is_continuous[Y, Z](g)
        implies is_continuous_at[X, Z](compose(g, f), x)
} by {
    if is_continuous_at[X, Y](f, x) and is_continuous[Y, Z](g) {
        continuous_imp_continuous_at[Y, Z](g, f(x))
        is_continuous_at[Y, Z](g, f(x))
        continuous_at_compose[X, Y, Z](f, g, x)
        is_continuous_at[X, Z](compose(g, f), x)
    }
}
