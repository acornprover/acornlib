from analysis.topology.topological_space import TopologicalSpace, is_continuous, compose,
    is_homeomorphism, is_homeomorphism_iff,
    is_homeomorphism_continuous_fwd, is_homeomorphism_continuous_inv,
    continuous_compose, homeomorphism_compose_left_inv,
    homeomorphism_compose_right_inv

/// Introduction rule for homeomorphisms from the four defining facts, isolated so
/// the definitional unfolding happens with a minimal premise pool.
theorem is_homeomorphism_intro[X: TopologicalSpace, Y: TopologicalSpace](
    f: X -> Y, g: Y -> X) {
    (is_continuous(f) and is_continuous(g)
        and (forall(x: X) { g(f(x)) = x })
        and (forall(y: Y) { f(g(y)) = y }))
        implies is_homeomorphism[X, Y](f, g)
} by {
    if (is_continuous(f) and is_continuous(g)
        and (forall(x: X) { g(f(x)) = x })
        and (forall(y: Y) { f(g(y)) = y })) {
        is_homeomorphism_iff[X, Y](f, g)
        is_homeomorphism[X, Y](f, g)
    }
}

/// The forward map of the composition of two homeomorphism pairs is continuous.
theorem homeomorphism_compose_continuous_fwd[X: TopologicalSpace, Y: TopologicalSpace, Z: TopologicalSpace](
    f1: X -> Y, g1: Y -> X, f2: Y -> Z, g2: Z -> Y) {
    is_homeomorphism[X, Y](f1, g1) and is_homeomorphism[Y, Z](f2, g2)
        implies is_continuous(compose(f2, f1))
} by {
    if is_homeomorphism[X, Y](f1, g1) and is_homeomorphism[Y, Z](f2, g2) {
        is_homeomorphism_continuous_fwd[X, Y](f1, g1)
        is_homeomorphism_continuous_fwd[Y, Z](f2, g2)
        (is_continuous(f1) and is_continuous(f2))
        continuous_compose[X, Y, Z](f1, f2)
        is_continuous(compose(f2, f1))
    }
}

/// The inverse map of the composition of two homeomorphism pairs is continuous.
theorem homeomorphism_compose_continuous_inv[X: TopologicalSpace, Y: TopologicalSpace, Z: TopologicalSpace](
    f1: X -> Y, g1: Y -> X, f2: Y -> Z, g2: Z -> Y) {
    is_homeomorphism[X, Y](f1, g1) and is_homeomorphism[Y, Z](f2, g2)
        implies is_continuous(compose(g1, g2))
} by {
    if is_homeomorphism[X, Y](f1, g1) and is_homeomorphism[Y, Z](f2, g2) {
        is_homeomorphism_continuous_inv[X, Y](f1, g1)
        is_homeomorphism_continuous_inv[Y, Z](f2, g2)
        (is_continuous(g2) and is_continuous(g1))
        continuous_compose[Z, Y, X](g2, g1)
        is_continuous(compose(g1, g2))
    }
}

/// The composed inverse is a left inverse of the composed forward map.
theorem homeomorphism_compose_left_inv_all[X: TopologicalSpace, Y: TopologicalSpace, Z: TopologicalSpace](
    f1: X -> Y, g1: Y -> X, f2: Y -> Z, g2: Z -> Y) {
    is_homeomorphism[X, Y](f1, g1) and is_homeomorphism[Y, Z](f2, g2)
        implies forall(x: X) { compose(g1, g2)(compose(f2, f1)(x)) = x }
} by {
    if is_homeomorphism[X, Y](f1, g1) and is_homeomorphism[Y, Z](f2, g2) {
        forall(x: X) {
            homeomorphism_compose_left_inv[X, Y, Z](f1, g1, f2, g2, x)
            compose(g1, g2)(compose(f2, f1)(x)) = x
        }
    }
}

/// The composed inverse is a right inverse of the composed forward map.
theorem homeomorphism_compose_right_inv_all[X: TopologicalSpace, Y: TopologicalSpace, Z: TopologicalSpace](
    f1: X -> Y, g1: Y -> X, f2: Y -> Z, g2: Z -> Y) {
    is_homeomorphism[X, Y](f1, g1) and is_homeomorphism[Y, Z](f2, g2)
        implies forall(z: Z) { compose(f2, f1)(compose(g1, g2)(z)) = z }
} by {
    if is_homeomorphism[X, Y](f1, g1) and is_homeomorphism[Y, Z](f2, g2) {
        forall(z: Z) {
            homeomorphism_compose_right_inv[X, Y, Z](f1, g1, f2, g2, z)
            compose(f2, f1)(compose(g1, g2)(z)) = z
        }
    }
}

/// The composition of two homeomorphism pairs is a homeomorphism pair.
theorem homeomorphism_compose[X: TopologicalSpace, Y: TopologicalSpace, Z: TopologicalSpace](
    f1: X -> Y, g1: Y -> X, f2: Y -> Z, g2: Z -> Y) {
    is_homeomorphism[X, Y](f1, g1) and is_homeomorphism[Y, Z](f2, g2)
        implies is_homeomorphism[X, Z](compose(f2, f1), compose(g1, g2))
} by {
    if is_homeomorphism[X, Y](f1, g1) and is_homeomorphism[Y, Z](f2, g2) {
        homeomorphism_compose_continuous_fwd[X, Y, Z](f1, g1, f2, g2)
        homeomorphism_compose_continuous_inv[X, Y, Z](f1, g1, f2, g2)
        homeomorphism_compose_left_inv_all[X, Y, Z](f1, g1, f2, g2)
        homeomorphism_compose_right_inv_all[X, Y, Z](f1, g1, f2, g2)
        is_homeomorphism_intro[X, Z](compose(f2, f1), compose(g1, g2))
        is_homeomorphism[X, Z](compose(f2, f1), compose(g1, g2))
    }
}
