from data.basic.set import Set, set_preimage, set_preimage_compl
from analysis.topology.topological_space import TopologicalSpace, is_continuous, is_closed,
    continuous_iff_preimage_open

/// The preimage of an open set under a continuous map is open.
theorem continuous_open_preimage[X: TopologicalSpace, Y: TopologicalSpace](
    f: X -> Y, v: Set[Y]) {
    is_continuous(f) and Y.is_open(v) implies X.is_open(set_preimage(f, v))
} by {
    if is_continuous(f) and Y.is_open(v) {
        continuous_iff_preimage_open(f)
        X.is_open(set_preimage(f, v))
    }
}

/// The preimage of a closed set under a continuous map is closed.
theorem continuous_closed_preimage[X: TopologicalSpace, Y: TopologicalSpace](
    f: X -> Y, c: Set[Y]) {
    is_continuous(f) and is_closed[Y](c) implies is_closed[X](set_preimage(f, c))
} by {
    if is_continuous(f) and is_closed[Y](c) {
        is_closed[Y](c) = Y.is_open(c.c)
        Y.is_open(c.c)
        continuous_open_preimage[X, Y](f, c.c)
        X.is_open(set_preimage(f, c.c))
        set_preimage_compl(f, c)
        set_preimage(f, c.c) = set_preimage(f, c).c
        X.is_open(set_preimage(f, c).c)
        is_closed[X](set_preimage(f, c)) = X.is_open(set_preimage(f, c).c)
        is_closed[X](set_preimage(f, c))
    }
}
