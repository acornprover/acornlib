from data.basic.set import Set, set_image, set_preimage, set_preimage_compl, set_preimage_empty,
    set_preimage_universal, set_preimage_eq_imp_eq_of_surjective,
    set_eq_transport_predicate, set_eq_transport_predicate_rev
from data.basic.functions import is_surjective_fn
from analysis.topology.topological_space import TopologicalSpace, is_continuous, is_compact,
    is_connected, is_connected_via_clopen, is_connected_via_clopen_iff,
    is_clopen, is_clopen_imp_open, is_clopen_intro,
    is_homeomorphism, is_homeomorphism_continuous_fwd,
    is_homeomorphism_continuous_inv
from analysis.topology.continuous_preimage import continuous_open_preimage
from analysis.topology.connectedness import connected_imp_via_clopen, via_clopen_imp_connected
from analysis.topology.compact_image import continuous_image_compact
from analysis.topology.homeomorphism_inverse import homeomorphism_inverse
from analysis.topology.homeomorphism_set_transport import homeomorphism_forward_surjective,
    homeomorphism_preimage_eq_inverse_image, homeomorphism_preimage_image_eq,
    homeomorphism_image_preimage_eq

/// The preimage of a clopen target set under a homeomorphism is clopen.
theorem homeomorphism_preimage_clopen[X: TopologicalSpace, Y: TopologicalSpace](
    f: X -> Y, g: Y -> X, t: Set[Y]) {
    is_homeomorphism[X, Y](f, g) and is_clopen[Y](Y.is_open, t)
        implies is_clopen[X](X.is_open, set_preimage(f, t))
} by {
    if is_homeomorphism[X, Y](f, g) and is_clopen[Y](Y.is_open, t) {
        is_clopen_imp_open[Y](Y.is_open, t)
        is_homeomorphism_continuous_fwd[X, Y](f, g)
        continuous_open_preimage[X, Y](f, t)
        continuous_open_preimage[X, Y](f, t.c)
        set_preimage_compl(f, t)
        let p: Set[X] -> Bool = function(u: Set[X]) { X.is_open(u) }
        set_eq_transport_predicate[X](p, set_preimage(f, t.c), set_preimage(f, t).c)
        X.is_open(set_preimage(f, t).c)
        is_clopen_intro[X](X.is_open, set_preimage(f, t))
        is_clopen[X](X.is_open, set_preimage(f, t))
    }
}

/// If a homeomorphism has empty preimage of a target set, then the target set is empty.
theorem homeomorphism_preimage_empty_imp_empty[X: TopologicalSpace, Y: TopologicalSpace](
    f: X -> Y, g: Y -> X, t: Set[Y]) {
    is_homeomorphism[X, Y](f, g) and set_preimage(f, t) = Set[X].empty_set
        implies t = Set[Y].empty_set
} by {
    if is_homeomorphism[X, Y](f, g) and set_preimage(f, t) = Set[X].empty_set {
        homeomorphism_forward_surjective[X, Y](f, g)
        set_preimage_empty[X, Y](f)
        set_preimage_eq_imp_eq_of_surjective[X, Y](t, Set[Y].empty_set, f)
        t = Set[Y].empty_set
    }
}

/// If a homeomorphism has universal preimage of a target set, then the target set is universal.
theorem homeomorphism_preimage_universal_imp_universal[X: TopologicalSpace, Y: TopologicalSpace](
    f: X -> Y, g: Y -> X, t: Set[Y]) {
    is_homeomorphism[X, Y](f, g) and set_preimage(f, t) = Set[X].universal_set
        implies t = Set[Y].universal_set
} by {
    if is_homeomorphism[X, Y](f, g) and set_preimage(f, t) = Set[X].universal_set {
        homeomorphism_forward_surjective[X, Y](f, g)
        set_preimage_universal[X, Y](f)
        set_preimage_eq_imp_eq_of_surjective[X, Y](t, Set[Y].universal_set, f)
        t = Set[Y].universal_set
    }
}

/// Homeomorphisms preserve the clopen-set characterization of connectedness.
theorem homeomorphism_preserves_connected_via_clopen[X: TopologicalSpace, Y: TopologicalSpace](
    f: X -> Y, g: Y -> X) {
    is_homeomorphism[X, Y](f, g) and is_connected_via_clopen[X](X.is_open)
        implies is_connected_via_clopen[Y](Y.is_open)
} by {
    if is_homeomorphism[X, Y](f, g) and is_connected_via_clopen[X](X.is_open) {
        is_connected_via_clopen_iff[X](X.is_open)
        is_connected_via_clopen_iff[Y](Y.is_open)
        forall(t: Set[Y]) {
            if is_clopen[Y](Y.is_open, t) {
                homeomorphism_preimage_clopen[X, Y](f, g, t)
                if set_preimage(f, t) = Set[X].empty_set {
                    homeomorphism_preimage_empty_imp_empty[X, Y](f, g, t)
                    t = Set[Y].empty_set or t = Set[Y].universal_set
                } else {
                    homeomorphism_preimage_universal_imp_universal[X, Y](f, g, t)
                    t = Set[Y].empty_set or t = Set[Y].universal_set
                }
                is_clopen[Y](Y.is_open, t) implies (t = Set[Y].empty_set or t = Set[Y].universal_set)
            }
        }
        forall(t: Set[Y]) {
            is_clopen[Y](Y.is_open, t) implies (t = Set[Y].empty_set or t = Set[Y].universal_set)
        }
    }
}

/// A homeomorphism sends compact source subsets to compact target subsets.
theorem homeomorphism_image_compact[X: TopologicalSpace, Y: TopologicalSpace](
    f: X -> Y, g: Y -> X, s: Set[X]) {
    is_homeomorphism[X, Y](f, g) and is_compact[X](s)
        implies is_compact[Y](set_image(s, f))
} by {
    if is_homeomorphism[X, Y](f, g) and is_compact[X](s) {
        is_homeomorphism_continuous_fwd[X, Y](f, g)
        is_continuous[X, Y](f)
        continuous_image_compact[X, Y](f, s)
        is_compact[Y](set_image(s, f))
    }
}

/// The forward preimage of a compact target subset under a homeomorphism is compact.
theorem homeomorphism_preimage_compact[X: TopologicalSpace, Y: TopologicalSpace](
    f: X -> Y, g: Y -> X, t: Set[Y]) {
    is_homeomorphism[X, Y](f, g) and is_compact[Y](t)
        implies is_compact[X](set_preimage(f, t))
} by {
    if is_homeomorphism[X, Y](f, g) and is_compact[Y](t) {
        is_homeomorphism_continuous_inv[X, Y](f, g)
        is_continuous[Y, X](g)
        continuous_image_compact[Y, X](g, t)
        is_compact[X](set_image(t, g))
        homeomorphism_preimage_eq_inverse_image[X, Y](f, g, t)
        set_preimage(f, t) = set_image(t, g)
        let p: Set[X] -> Bool = function(u: Set[X]) { is_compact[X](u) }
        set_eq_transport_predicate_rev[X](p, set_preimage(f, t), set_image(t, g))
        is_compact[X](set_preimage(f, t))
    }
}

/// A source subset is compact exactly when its forward homeomorphic image is compact.
theorem homeomorphism_image_compact_iff[X: TopologicalSpace, Y: TopologicalSpace](
    f: X -> Y, g: Y -> X, s: Set[X]) {
    is_homeomorphism[X, Y](f, g) implies
        (is_compact[X](s) = is_compact[Y](set_image(s, f)))
} by {
    if is_homeomorphism[X, Y](f, g) {
        if is_compact[X](s) {
            homeomorphism_image_compact[X, Y](f, g, s)
            is_compact[Y](set_image(s, f))
        }
        if is_compact[Y](set_image(s, f)) {
            homeomorphism_preimage_compact[X, Y](f, g, set_image(s, f))
            is_compact[X](set_preimage(f, set_image(s, f)))
            homeomorphism_preimage_image_eq[X, Y](f, g, s)
            set_preimage(f, set_image(s, f)) = s
            let p: Set[X] -> Bool = function(u: Set[X]) { is_compact[X](u) }
            set_eq_transport_predicate[X](p, set_preimage(f, set_image(s, f)), s)
            is_compact[X](s)
        }
        is_compact[X](s) = is_compact[Y](set_image(s, f))
    }
}

/// A target subset is compact exactly when its forward preimage under a homeomorphism is compact.
theorem homeomorphism_preimage_compact_iff[X: TopologicalSpace, Y: TopologicalSpace](
    f: X -> Y, g: Y -> X, t: Set[Y]) {
    is_homeomorphism[X, Y](f, g) implies
        (is_compact[Y](t) = is_compact[X](set_preimage(f, t)))
} by {
    if is_homeomorphism[X, Y](f, g) {
        if is_compact[Y](t) {
            homeomorphism_preimage_compact[X, Y](f, g, t)
            is_compact[X](set_preimage(f, t))
        }
        if is_compact[X](set_preimage(f, t)) {
            homeomorphism_image_compact[X, Y](f, g, set_preimage(f, t))
            is_compact[Y](set_image(set_preimage(f, t), f))
            homeomorphism_image_preimage_eq[X, Y](f, g, t)
            set_image(set_preimage(f, t), f) = t
            let p: Set[Y] -> Bool = function(u: Set[Y]) { is_compact[Y](u) }
            set_eq_transport_predicate[Y](p, set_image(set_preimage(f, t), f), t)
            is_compact[Y](t)
        }
        is_compact[Y](t) = is_compact[X](set_preimage(f, t))
    }
}

/// A homeomorphism preserves connectedness of the whole space.
theorem homeomorphism_preserves_connected[X: TopologicalSpace, Y: TopologicalSpace](
    f: X -> Y, g: Y -> X) {
    is_homeomorphism[X, Y](f, g) and is_connected[X](X.is_open)
        implies is_connected[Y](Y.is_open)
} by {
    if is_homeomorphism[X, Y](f, g) and is_connected[X](X.is_open) {
        connected_imp_via_clopen[X](X.is_open)
        homeomorphism_preserves_connected_via_clopen[X, Y](f, g)
        via_clopen_imp_connected[Y](Y.is_open)
        is_connected[Y](Y.is_open)
    }
}

/// A homeomorphism reflects connectedness of the whole space through its inverse.
theorem homeomorphism_reflects_connected[X: TopologicalSpace, Y: TopologicalSpace](
    f: X -> Y, g: Y -> X) {
    is_homeomorphism[X, Y](f, g) and is_connected[Y](Y.is_open)
        implies is_connected[X](X.is_open)
} by {
    if is_homeomorphism[X, Y](f, g) and is_connected[Y](Y.is_open) {
        homeomorphism_inverse[X, Y](f, g)
        is_homeomorphism[Y, X](g, f)
        homeomorphism_preserves_connected[Y, X](g, f)
        is_connected[X](X.is_open)
    }
}

/// Homeomorphic spaces are connected simultaneously.
theorem homeomorphism_connected_iff[X: TopologicalSpace, Y: TopologicalSpace](
    f: X -> Y, g: Y -> X) {
    is_homeomorphism[X, Y](f, g) implies
        (is_connected[X](X.is_open) = is_connected[Y](Y.is_open))
} by {
    if is_homeomorphism[X, Y](f, g) {
        if is_connected[X](X.is_open) {
            homeomorphism_preserves_connected[X, Y](f, g)
            is_connected[Y](Y.is_open)
        }
        if is_connected[Y](Y.is_open) {
            homeomorphism_reflects_connected[X, Y](f, g)
            is_connected[X](X.is_open)
        }
        is_connected[X](X.is_open) = is_connected[Y](Y.is_open)
    }
}
