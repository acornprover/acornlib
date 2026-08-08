/// Foundations of measure theory: outer measures and Carathéodory measurability.
///
/// An outer measure on a type `K` is a set function taking values in the
/// reals.  The library has no extended reals, so `infinity` is not
/// representable; accordingly countable subadditivity is expressed here in
/// its finite form (subadditivity over a finite union of a family), and the
/// full countable statement is documented as a comment below.  The
/// Carathéodory construction is developed for the finite (binary) closure
/// operations; closure under countable union is the classical hard part and
/// is left as a documented TODO.
from data.basic.set import Set, set_ext, union_contains_eq, union_contains_cases,
    union_contains_left, union_comm, intersection_contains_eq,
    difference_contains_eq, compl_contains_eq, empty_set_contains_eq,
    universal_set_contains_eq, intersection_with_universal_is_self,
    intersection_with_empty_is_empty,
    range_indexed_union, range_indexed_union_contains_eq,
    range_indexed_union_contains_of_lt, range_indexed_union_contains_witness,
    range_indexed_union_zero
from data.basic.functions import compose
from nat import Nat, lt_suc, lt_suc_right, lt_imp_lt_suc
from real import Real, lte_trans, lte_trans_eq
from list import partial, partial_split_last

numerals Real

/// True if `m` is an outer measure on `K`: `m` is nonnegative, vanishes on
/// the empty set, is monotone under inclusion, and is finitely subadditive.
define is_outer_measure[K](m: Set[K] -> Real) -> Bool {
    (forall(s: Set[K]) { Real.0 <= m(s) })
        and m(Set[K].empty_set) = Real.0
        and (forall(a: Set[K], b: Set[K]) { a.subset(b) implies m(a) <= m(b) })
        and (forall(a: Set[K], b: Set[K]) { m(a.union(b)) <= m(a) + m(b) })
}

// Countable subadditivity (the classical fourth axiom) states that for every
// family `family: Nat -> Set[K]`,
//     m(indexed_union(family)) <= sum_{n : Nat} m(family(n))
// where the right hand side is the (possibly infinite) sum of the nonnegative
// terms.  Since the library's reals have no element `infinity`, the countable
// sum is not directly representable; the finite subadditivity axiom above,
// together with `outer_measure_finite_subadditive` below, gives the finite
// approximation `m(range_indexed_union(n, family)) <= partial(measure_seq(m, family), n)`
// for every bound `n`.  A full formulation would require an extended-real
// codomain or an explicit convergence hypothesis on the partial sums.

/// A structure bundling a set function with the outer measure axioms.
structure OuterMeasure[K] {
    /// The outer measure value of a set.
    measure: Set[K] -> Real
} constraint {
    is_outer_measure(measure)
}

/// The outer measure axioms of `m`, unpacked.
theorem outer_measure_constraint[K](m: OuterMeasure[K]) {
    ((forall(s: Set[K]) { Real.0 <= m.measure(s) })
        and m.measure(Set[K].empty_set) = Real.0
        and (forall(u: Set[K], v: Set[K]) { u.subset(v) implies m.measure(u) <= m.measure(v) })
        and (forall(u: Set[K], v: Set[K]) { m.measure(u.union(v)) <= m.measure(u) + m.measure(v) }))
} by {
    is_outer_measure(m.measure) =
        ((forall(s: Set[K]) { Real.0 <= m.measure(s) })
            and m.measure(Set[K].empty_set) = Real.0
            and (forall(u: Set[K], v: Set[K]) { u.subset(v) implies m.measure(u) <= m.measure(v) })
            and (forall(u: Set[K], v: Set[K]) { m.measure(u.union(v)) <= m.measure(u) + m.measure(v) }))
    is_outer_measure(m.measure)
    ((forall(s: Set[K]) { Real.0 <= m.measure(s) })
        and m.measure(Set[K].empty_set) = Real.0
        and (forall(u: Set[K], v: Set[K]) { u.subset(v) implies m.measure(u) <= m.measure(v) })
        and (forall(u: Set[K], v: Set[K]) { m.measure(u.union(v)) <= m.measure(u) + m.measure(v) }))
}

/// The outer measure of every set is nonnegative.
theorem outer_measure_nonneg[K](m: OuterMeasure[K], s: Set[K]) {
    Real.0 <= m.measure(s)
} by {
    outer_measure_constraint(m)
    forall(z: Set[K]) { Real.0 <= m.measure(z) }
}

/// The outer measure of the empty set is zero.
theorem outer_measure_empty[K](m: OuterMeasure[K]) {
    m.measure(Set[K].empty_set) = Real.0
} by {
    outer_measure_constraint(m)
    m.measure(Set[K].empty_set) = Real.0
}

/// Outer measures are monotone: a subset has no larger measure.
theorem outer_measure_monotone[K](m: OuterMeasure[K], a: Set[K], b: Set[K]) {
    a.subset(b) implies m.measure(a) <= m.measure(b)
} by {
    if a.subset(b) {
        outer_measure_constraint(m)
        forall(u: Set[K], v: Set[K]) { u.subset(v) implies m.measure(u) <= m.measure(v) }
        m.measure(a) <= m.measure(b)
    }
}

/// Outer measures are finitely (binary) subadditive.
theorem outer_measure_subadditive[K](m: OuterMeasure[K], a: Set[K], b: Set[K]) {
    m.measure(a.union(b)) <= m.measure(a) + m.measure(b)
} by {
    outer_measure_constraint(m)
    forall(u: Set[K], v: Set[K]) { m.measure(u.union(v)) <= m.measure(u) + m.measure(v) }
}

/// A set is contained in its union with any other set.
theorem subset_of_union_left[K](a: Set[K], b: Set[K]) {
    a.subset(a.union(b))
} by {
    forall(x: K) {
        if a.contains(x) {
            union_contains_left(a, b, x)
            a.union(b).contains(x)
        }
    }
}

/// The measure of a set is at most the measure of its union with any set.
theorem outer_measure_monotone_union[K](m: OuterMeasure[K], a: Set[K], b: Set[K]) {
    m.measure(a) <= m.measure(a.union(b))
} by {
    subset_of_union_left(a, b)
    outer_measure_monotone(m, a, a.union(b))
}

/// A set difference is contained in its first operand.
theorem difference_subset_left[K](a: Set[K], b: Set[K]) {
    a.difference(b).subset(a)
} by {
    forall(x: K) {
        if a.difference(b).contains(x) {
            difference_contains_eq(a, b, x)
            a.contains(x)
        }
    }
}

/// The measure of a difference is at most the measure of its first operand.
theorem outer_measure_monotone_difference[K](m: OuterMeasure[K], a: Set[K], b: Set[K]) {
    m.measure(a.difference(b)) <= m.measure(a)
} by {
    difference_subset_left(a, b)
    outer_measure_monotone(m, a.difference(b), a)
}

/// The sequence of outer measures of the members of a family.
define measure_seq[K](m: OuterMeasure[K], family: Nat -> Set[K], n: Nat) -> Real {
    m.measure(family(n))
}

/// The partial sum of the measures of the first `n` members of a family.
define range_measure[K](m: OuterMeasure[K], family: Nat -> Set[K], n: Nat) -> Real {
    partial(measure_seq(m, family), n)
}

/// The partial sum over an empty range is zero.
theorem range_measure_zero[K](m: OuterMeasure[K], family: Nat -> Set[K]) {
    range_measure(m, family, Nat.0) = Real.0
} by {
    range_measure(m, family, Nat.0) = partial(measure_seq(m, family), Nat.0)
    partial(measure_seq(m, family), Nat.0) = Real.0
}

/// The partial sum up to `k.suc` splits off the last term.
theorem range_measure_suc[K](m: OuterMeasure[K], family: Nat -> Set[K], k: Nat) {
    range_measure(m, family, k.suc) = range_measure(m, family, k) + measure_seq(m, family, k)
} by {
    range_measure(m, family, k.suc) = partial(measure_seq(m, family), k.suc)
    partial_split_last(measure_seq(m, family), k)
    partial(measure_seq(m, family), k.suc) = partial(measure_seq(m, family), k) + measure_seq(m, family, k)
    range_measure(m, family, k) = partial(measure_seq(m, family), k)
    range_measure(m, family, k.suc) = range_measure(m, family, k) + measure_seq(m, family, k)
}

/// The union of the first `n.suc` members of a family is the union of the
/// first `n` members with the next member.
theorem range_indexed_union_suc[K](family: Nat -> Set[K], n: Nat) {
    range_indexed_union(n.suc, family) = range_indexed_union(n, family).union(family(n))
} by {
    forall(x: K) {
        range_indexed_union_contains_eq(n.suc, family, x)
        range_indexed_union_contains_eq(n, family, x)
        union_contains_eq(range_indexed_union(n, family), family(n), x)
        if range_indexed_union(n.suc, family).contains(x) {
            range_indexed_union_contains_witness(n.suc, family, x)
            let i: Nat satisfy { i < n.suc and family(i).contains(x) }
            lt_suc_right(i, n)
            if i = n {
                family(n).contains(x)
                union_contains_eq(range_indexed_union(n, family), family(n), x)
                range_indexed_union(n, family).union(family(n)).contains(x)
            }
            if i < n {
                range_indexed_union_contains_of_lt(n, family, i, x)
                range_indexed_union(n, family).contains(x)
                union_contains_eq(range_indexed_union(n, family), family(n), x)
                range_indexed_union(n, family).union(family(n)).contains(x)
            }
            range_indexed_union(n, family).union(family(n)).contains(x)
        }
        if range_indexed_union(n, family).union(family(n)).contains(x) {
            union_contains_cases(range_indexed_union(n, family), family(n), x)
            if range_indexed_union(n, family).contains(x) {
                range_indexed_union_contains_witness(n, family, x)
                let i: Nat satisfy { i < n and family(i).contains(x) }
                lt_imp_lt_suc(i, n)
                i < n.suc
                range_indexed_union_contains_of_lt(n.suc, family, i, x)
                range_indexed_union(n.suc, family).contains(x)
            }
            if family(n).contains(x) {
                lt_suc(n)
                n < n.suc
                range_indexed_union_contains_of_lt(n.suc, family, n, x)
                range_indexed_union(n.suc, family).contains(x)
            }
            range_indexed_union(n.suc, family).contains(x)
        }
        range_indexed_union(n.suc, family).contains(x) =
            range_indexed_union(n, family).union(family(n)).contains(x)
    }
    set_ext(range_indexed_union(n.suc, family), range_indexed_union(n, family).union(family(n)))
}

/// Finite subadditivity holds for every bound: the measure of the union of
/// the first `n` members of a family is at most the partial sum of their
/// measures.
define finite_subadd_hyp[K](m: OuterMeasure[K], family: Nat -> Set[K], n: Nat) -> Bool {
    m.measure(range_indexed_union(n, family)) <= range_measure(m, family, n)
}

/// A member of the finite subadditivity hypothesis yields the raw inequality.
theorem finite_subadd_hyp_apply[K](m: OuterMeasure[K], family: Nat -> Set[K], k: Nat) {
    finite_subadd_hyp(m, family, k) implies
        m.measure(range_indexed_union(k, family)) <= range_measure(m, family, k)
} by {
    if finite_subadd_hyp(m, family, k) {
        m.measure(range_indexed_union(k, family)) <= range_measure(m, family, k)
    }
}

/// The raw inequality yields a member of the finite subadditivity hypothesis.
theorem finite_subadd_hyp_intro[K](m: OuterMeasure[K], family: Nat -> Set[K], k: Nat) {
    (m.measure(range_indexed_union(k, family)) <= range_measure(m, family, k))
        implies finite_subadd_hyp(m, family, k)
} by {
    if m.measure(range_indexed_union(k, family)) <= range_measure(m, family, k) {
        finite_subadd_hyp(m, family, k)
    }
}

/// Finite subadditivity, quantified over the bound.
theorem outer_measure_finite_subadditive_all[K](m: OuterMeasure[K],
        family: Nat -> Set[K]) {
    forall(n: Nat) { finite_subadd_hyp(m, family, n) }
} by {
    define p(k: Nat) -> Bool {
        finite_subadd_hyp(m, family, k)
    }
    range_indexed_union_zero(family)
    range_indexed_union(Nat.0, family) = Set[K].empty_set
    outer_measure_empty(m)
    m.measure(range_indexed_union(Nat.0, family)) = Real.0
    range_measure_zero(m, family)
    range_measure(m, family, Nat.0) = Real.0
    m.measure(range_indexed_union(Nat.0, family)) <= range_measure(m, family, Nat.0)
    finite_subadd_hyp_intro(m, family, Nat.0)
    finite_subadd_hyp(m, family, Nat.0)
    p(Nat.0)
    forall(k: Nat) {
        if p(k) {
            p(k) = finite_subadd_hyp(m, family, k)
            finite_subadd_hyp(m, family, k)
            finite_subadd_hyp_apply(m, family, k)
            m.measure(range_indexed_union(k, family)) <= range_measure(m, family, k)
            range_indexed_union_suc(family, k)
            m.measure(range_indexed_union(k.suc, family)) <= m.measure(range_indexed_union(k, family).union(family(k)))
            outer_measure_subadditive(m, range_indexed_union(k, family), family(k))
            m.measure(range_indexed_union(k.suc, family)) <= m.measure(range_indexed_union(k, family)) + m.measure(family(k))
            m.measure(range_indexed_union(k, family)) + m.measure(family(k)) <= range_measure(m, family, k) + m.measure(family(k))
            lte_trans(m.measure(range_indexed_union(k.suc, family)), m.measure(range_indexed_union(k, family)) + m.measure(family(k)), range_measure(m, family, k) + m.measure(family(k)))
            m.measure(range_indexed_union(k.suc, family)) <= range_measure(m, family, k) + m.measure(family(k))
            measure_seq(m, family, k) = m.measure(family(k))
            range_measure(m, family, k) + m.measure(family(k)) = range_measure(m, family, k) + measure_seq(m, family, k)
            m.measure(range_indexed_union(k.suc, family)) <= range_measure(m, family, k) + measure_seq(m, family, k)
            range_measure_suc(m, family, k)
            range_measure(m, family, k.suc) = range_measure(m, family, k) + measure_seq(m, family, k)
            lte_trans_eq(m.measure(range_indexed_union(k.suc, family)), range_measure(m, family, k) + measure_seq(m, family, k), range_measure(m, family, k.suc))
            m.measure(range_indexed_union(k.suc, family)) <= range_measure(m, family, k.suc)
            finite_subadd_hyp_intro(m, family, k.suc)
            finite_subadd_hyp(m, family, k.suc)
            p(k.suc)
        }
    }
}

/// Finite subadditivity at a fixed bound.
theorem outer_measure_finite_subadditive[K](m: OuterMeasure[K],
        family: Nat -> Set[K], n: Nat) {
    m.measure(range_indexed_union(n, family)) <= range_measure(m, family, n)
} by {
    outer_measure_finite_subadditive_all(m, family)
    forall(i: Nat) { finite_subadd_hyp(m, family, i) }
    finite_subadd_hyp_apply(m, family, n)
    m.measure(range_indexed_union(n, family)) <= range_measure(m, family, n)
}

/// A set `a` is Carathéodory measurable with respect to the outer measure
/// `m` when `m` splits every test set `e` at `a`:
/// `m(e) = m(e ∩ a) + m(e \ a)`.
define caratheodory_measurable[K](m: OuterMeasure[K], a: Set[K]) -> Bool {
    forall(e: Set[K]) {
        m.measure(e) = m.measure(e.intersection(a)) + m.measure(e.difference(a))
    }
}

/// The intersection of a set with the universal set is the set itself.
theorem intersection_with_universal_self[K](e: Set[K]) {
    e.intersection(Set[K].universal_set) = e
} by {
    intersection_with_universal_is_self(e)
}

/// The difference of a set with the universal set is empty.
theorem difference_universal_is_empty[K](e: Set[K]) {
    e.difference(Set[K].universal_set) = Set[K].empty_set
} by {
    forall(x: K) {
        difference_contains_eq(e, Set[K].universal_set, x)
        universal_set_contains_eq[K](x)
        if e.difference(Set[K].universal_set).contains(x) {
            empty_set_contains_eq[K](x)
        }
        e.difference(Set[K].universal_set).contains(x) = Set[K].empty_set.contains(x)
    }
    set_ext(e.difference(Set[K].universal_set), Set[K].empty_set)
}

/// The difference of a set with the empty set is the set itself.
theorem difference_empty_is_self[K](e: Set[K]) {
    e.difference(Set[K].empty_set) = e
} by {
    forall(x: K) {
        difference_contains_eq(e, Set[K].empty_set, x)
        empty_set_contains_eq[K](x)
        if e.difference(Set[K].empty_set).contains(x) {
            e.contains(x)
        }
        if e.contains(x) {
            empty_set_contains_eq[K](x)
            difference_contains_eq(e, Set[K].empty_set, x)
            e.difference(Set[K].empty_set).contains(x)
        }
        e.difference(Set[K].empty_set).contains(x) = e.contains(x)
    }
    set_ext(e.difference(Set[K].empty_set), e)
}

/// The universal set is Carathéodory measurable.
theorem caratheodory_measurable_universal[K](m: OuterMeasure[K]) {
    caratheodory_measurable(m, Set[K].universal_set)
} by {
    forall(e: Set[K]) {
        intersection_with_universal_self(e)
        e.intersection(Set[K].universal_set) = e
        difference_universal_is_empty(e)
        e.difference(Set[K].universal_set) = Set[K].empty_set
        outer_measure_empty(m)
        m.measure(Set[K].empty_set) = Real.0
        m.measure(e) = m.measure(e.intersection(Set[K].universal_set)) + m.measure(e.difference(Set[K].universal_set))
    }
}

/// The empty set is Carathéodory measurable.
theorem caratheodory_measurable_empty[K](m: OuterMeasure[K]) {
    caratheodory_measurable(m, Set[K].empty_set)
} by {
    forall(e: Set[K]) {
        intersection_with_empty_is_empty(e)
        e.intersection(Set[K].empty_set) = Set[K].empty_set
        difference_empty_is_self(e)
        e.difference(Set[K].empty_set) = e
        outer_measure_empty(m)
        m.measure(Set[K].empty_set) = Real.0
        m.measure(e) = m.measure(e.intersection(Set[K].empty_set)) + m.measure(e.difference(Set[K].empty_set))
    }
}

/// The intersection of a set with a complement is the difference.
theorem intersection_compl_is_difference[K](e: Set[K], a: Set[K]) {
    e.intersection(a.c) = e.difference(a)
} by {
    forall(x: K) {
        intersection_contains_eq(e, a.c, x)
        compl_contains_eq(a, x)
        difference_contains_eq(e, a, x)
        if e.intersection(a.c).contains(x) {
            e.difference(a).contains(x)
        }
        if e.difference(a).contains(x) {
            e.intersection(a.c).contains(x)
        }
        e.intersection(a.c).contains(x) = e.difference(a).contains(x)
    }
    set_ext(e.intersection(a.c), e.difference(a))
}

/// The difference of a set with a complement is the intersection.
theorem difference_compl_is_intersection[K](e: Set[K], a: Set[K]) {
    e.difference(a.c) = e.intersection(a)
} by {
    forall(x: K) {
        difference_contains_eq(e, a.c, x)
        compl_contains_eq(a, x)
        intersection_contains_eq(e, a, x)
        if e.difference(a.c).contains(x) {
            e.intersection(a).contains(x)
        }
        if e.intersection(a).contains(x) {
            e.difference(a.c).contains(x)
        }
        e.difference(a.c).contains(x) = e.intersection(a).contains(x)
    }
    set_ext(e.difference(a.c), e.intersection(a))
}

/// Carathéodory measurability is closed under complement.
theorem caratheodory_measurable_compl[K](m: OuterMeasure[K], a: Set[K]) {
    caratheodory_measurable(m, a) implies caratheodory_measurable(m, a.c)
} by {
    if caratheodory_measurable(m, a) {
        forall(e: Set[K]) {
            caratheodory_measurable(m, a) = forall(f: Set[K]) {
                m.measure(f) = m.measure(f.intersection(a)) + m.measure(f.difference(a))
            }
            m.measure(e) = m.measure(e.intersection(a)) + m.measure(e.difference(a))
            intersection_compl_is_difference(e, a)
            e.intersection(a.c) = e.difference(a)
            difference_compl_is_intersection(e, a)
            e.difference(a.c) = e.intersection(a)
            m.measure(e) = m.measure(e.intersection(a.c)) + m.measure(e.difference(a.c))
        }
    }
}

/// The iterated difference `(e \ b) \ a` equals `e \ (a ∪ b)`.
theorem difference_iterated_union[K](e: Set[K], a: Set[K], b: Set[K]) {
    (e.difference(b)).difference(a) = e.difference(a.union(b))
} by {
    forall(x: K) {
        difference_contains_eq(e.difference(b), a, x)
        difference_contains_eq(e, b, x)
        difference_contains_eq(e, a.union(b), x)
        union_contains_eq(a, b, x)
        if (e.difference(b)).difference(a).contains(x) {
            e.difference(a.union(b)).contains(x)
        }
        if e.difference(a.union(b)).contains(x) {
            (e.difference(b)).difference(a).contains(x)
        }
        (e.difference(b)).difference(a).contains(x) = e.difference(a.union(b)).contains(x)
    }
    set_ext((e.difference(b)).difference(a), e.difference(a.union(b)))
}

/// The intersection of `e ∩ (a ∪ b)` with `a` is `e ∩ a`.
theorem intersection_union_intersect_left[K](e: Set[K], a: Set[K], b: Set[K]) {
    (e.intersection(a.union(b))).intersection(a) = e.intersection(a)
} by {
    forall(x: K) {
        intersection_contains_eq(e.intersection(a.union(b)), a, x)
        intersection_contains_eq(e, a.union(b), x)
        union_contains_eq(a, b, x)
        intersection_contains_eq(e, a, x)
        if (e.intersection(a.union(b))).intersection(a).contains(x) {
            e.intersection(a).contains(x)
        }
        if e.intersection(a).contains(x) {
            (e.intersection(a.union(b))).intersection(a).contains(x)
        }
        (e.intersection(a.union(b))).intersection(a).contains(x) = e.intersection(a).contains(x)
    }
    set_ext((e.intersection(a.union(b))).intersection(a), e.intersection(a))
}

/// The difference of `e ∩ (a ∪ b)` with `a` is `(e \ a) ∩ b`.
theorem intersection_union_difference_left[K](e: Set[K], a: Set[K], b: Set[K]) {
    (e.intersection(a.union(b))).difference(a) = (e.difference(a)).intersection(b)
} by {
    forall(x: K) {
        difference_contains_eq(e.intersection(a.union(b)), a, x)
        intersection_contains_eq(e, a.union(b), x)
        union_contains_eq(a, b, x)
        difference_contains_eq(e, a, x)
        intersection_contains_eq(e.difference(a), b, x)
        if (e.intersection(a.union(b))).difference(a).contains(x) {
            (e.difference(a)).intersection(b).contains(x)
        }
        if (e.difference(a)).intersection(b).contains(x) {
            (e.intersection(a.union(b))).difference(a).contains(x)
        }
        (e.intersection(a.union(b))).difference(a).contains(x) =
            (e.difference(a)).intersection(b).contains(x)
    }
    set_ext((e.intersection(a.union(b))).difference(a), (e.difference(a)).intersection(b))
}

/// Carathéodory measurability is closed under binary union: if `a` and `b`
/// are measurable, then so is `a ∪ b`.
theorem caratheodory_measurable_union[K](m: OuterMeasure[K], a: Set[K], b: Set[K]) {
    caratheodory_measurable(m, a) and caratheodory_measurable(m, b)
        implies caratheodory_measurable(m, a.union(b))
} by {
    if caratheodory_measurable(m, a) and caratheodory_measurable(m, b) {
        caratheodory_measurable(m, a) = forall(f: Set[K]) {
            m.measure(f) = m.measure(f.intersection(a)) + m.measure(f.difference(a))
        }
        caratheodory_measurable(m, b) = forall(g: Set[K]) {
            m.measure(g) = m.measure(g.intersection(b)) + m.measure(g.difference(b))
        }
        forall(e: Set[K]) {
            m.measure(e) = m.measure(e.intersection(a)) + m.measure(e.difference(a))
            m.measure(e.intersection(a)) = m.measure((e.intersection(a)).intersection(b)) + m.measure((e.intersection(a)).difference(b))
            m.measure(e.difference(a)) = m.measure((e.difference(a)).intersection(b)) + m.measure((e.difference(a)).difference(b))
            m.measure(e) = m.measure((e.intersection(a)).intersection(b)) + m.measure((e.intersection(a)).difference(b)) + m.measure((e.difference(a)).intersection(b)) + m.measure((e.difference(a)).difference(b))
            difference_iterated_union(e, b, a)
            (e.difference(a)).difference(b) = e.difference(b.union(a))
            union_comm(b, a)
            b.union(a) = a.union(b)
            (e.difference(a)).difference(b) = e.difference(a.union(b))
            m.measure(e) = m.measure((e.intersection(a)).intersection(b)) + m.measure((e.intersection(a)).difference(b)) + m.measure((e.difference(a)).intersection(b)) + m.measure(e.difference(a.union(b)))
            m.measure(e.intersection(a.union(b))) = m.measure((e.intersection(a.union(b))).intersection(a)) + m.measure((e.intersection(a.union(b))).difference(a))
            intersection_union_intersect_left(e, a, b)
            (e.intersection(a.union(b))).intersection(a) = e.intersection(a)
            intersection_union_difference_left(e, a, b)
            (e.intersection(a.union(b))).difference(a) = (e.difference(a)).intersection(b)
            m.measure(e.intersection(a.union(b))) = m.measure(e.intersection(a)) + m.measure((e.difference(a)).intersection(b))
            m.measure(e.intersection(a)) = m.measure((e.intersection(a)).intersection(b)) + m.measure((e.intersection(a)).difference(b))
            m.measure(e.intersection(a.union(b))) = m.measure((e.intersection(a)).intersection(b)) + m.measure((e.intersection(a)).difference(b)) + m.measure((e.difference(a)).intersection(b))
            m.measure((e.intersection(a)).intersection(b)) + m.measure((e.intersection(a)).difference(b)) + m.measure((e.difference(a)).intersection(b)) + m.measure(e.difference(a.union(b))) = m.measure(e.intersection(a.union(b))) + m.measure(e.difference(a.union(b)))
            m.measure(e) = m.measure(e.intersection(a.union(b))) + m.measure(e.difference(a.union(b)))
        }
    }
}

/// The complement of a binary union is the intersection of the complements
/// (De Morgan's law).
theorem union_compl_is_intersection_compl[K](a: Set[K], b: Set[K]) {
    (a.c.union(b.c)).c = a.intersection(b)
} by {
    forall(x: K) {
        compl_contains_eq(a.c.union(b.c), x)
        union_contains_eq(a.c, b.c, x)
        compl_contains_eq(a, x)
        compl_contains_eq(b, x)
        intersection_contains_eq(a, b, x)
        if a.intersection(b).contains(x) {
            (a.c.union(b.c)).c.contains(x)
        }
        if (a.c.union(b.c)).c.contains(x) {
            a.intersection(b).contains(x)
        }
        (a.c.union(b.c)).c.contains(x) = a.intersection(b).contains(x)
    }
    set_ext((a.c.union(b.c)).c, a.intersection(b))
}

/// Carathéodory measurability is closed under binary intersection: if `a`
/// and `b` are measurable, then so is `a ∩ b`.
theorem caratheodory_measurable_intersection[K](m: OuterMeasure[K], a: Set[K], b: Set[K]) {
    caratheodory_measurable(m, a) and caratheodory_measurable(m, b)
        implies caratheodory_measurable(m, a.intersection(b))
} by {
    if caratheodory_measurable(m, a) and caratheodory_measurable(m, b) {
        caratheodory_measurable_compl(m, a)
        caratheodory_measurable_compl(m, b)
        caratheodory_measurable_union(m, a.c, b.c)
        caratheodory_measurable(m, a.c.union(b.c))
        union_compl_is_intersection_compl(a, b)
        (a.c.union(b.c)).c = a.intersection(b)
        caratheodory_measurable_compl(m, a.c.union(b.c))
        caratheodory_measurable(m, a.intersection(b))
    }
}

/// A set difference equals the intersection with the complement.
theorem difference_eq_intersection_compl[K](a: Set[K], b: Set[K]) {
    a.difference(b) = a.intersection(b.c)
} by {
    forall(x: K) {
        difference_contains_eq(a, b, x)
        intersection_contains_eq(a, b.c, x)
        compl_contains_eq(b, x)
        if a.difference(b).contains(x) {
            a.intersection(b.c).contains(x)
        }
        if a.intersection(b.c).contains(x) {
            a.difference(b).contains(x)
        }
        a.difference(b).contains(x) = a.intersection(b.c).contains(x)
    }
    set_ext(a.difference(b), a.intersection(b.c))
}

/// Carathéodory measurability is closed under set difference: if `a` and `b`
/// are measurable, then so is `a \ b`.
theorem caratheodory_measurable_difference[K](m: OuterMeasure[K], a: Set[K], b: Set[K]) {
    caratheodory_measurable(m, a) and caratheodory_measurable(m, b)
        implies caratheodory_measurable(m, a.difference(b))
} by {
    if caratheodory_measurable(m, a) and caratheodory_measurable(m, b) {
        caratheodory_measurable_compl(m, b)
        caratheodory_measurable(m, b.c)
        caratheodory_measurable_intersection(m, a, b.c)
        caratheodory_measurable(m, a.intersection(b.c))
        difference_eq_intersection_compl(a, b)
        a.difference(b) = a.intersection(b.c)
        caratheodory_measurable(m, a.difference(b))
    }
}

// TODO: the classical Carathéodory theorem states that the collection of
// Carathéodory measurable sets is a sigma-algebra, i.e. it is additionally
// closed under countable unions.  The standard proof disjointifies a countable
// family (B_n = A_n \ ⋃_{i < n} A_i), shows m(E ∩ ⋃_{n <= N} B_n) is the
// partial sum of the m(E ∩ B_n) by induction, and passes to the limit using
// countable subadditivity and the monotone convergence principle for the
// nonnegative partial sums.  In the current Real-valued setting the limit
// step needs the partial sums to converge, which the library's real_series
// machinery supports via `converges_to(partial(...), l)`, but the full proof
// is deferred.  The finite closure results above (empty, universal,
// complement, binary union, binary intersection, difference) are the building
// blocks; a `SigmaAlgebra[K]` instance
//     SigmaAlgebra.new(function(s: Set[K]) { caratheodory_measurable(m, s) })
// would follow once `closed_under_countable_union_constraint` is proved.
