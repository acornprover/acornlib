/// Compactness closure properties in Hausdorff spaces.
///
/// This file completes the compactness story of the general topology
/// framework: a compact subset of a Hausdorff space is closed, and the union
/// of the sets of a finite list of compact sets is compact.  The main proof
/// separates a point `x` outside a compact set `s` from `s` by an open
/// neighborhood: for each point `y` of `s`, Hausdorff separation provides
/// disjoint open neighborhoods `u_y` of `x` and `v_y` of `y`; the `v_y`'s
/// cover `s`, so compactness yields a finite subcover, and the finite
/// intersection of the corresponding `u_y`'s is an open neighborhood of `x`
/// disjoint from `s`.

from list import List
from data.list.list_cons_membership import cons_contains_eq, cons_contains_head,
    cons_contains_of_tail_contains, cons_contains_cases, nil_not_contains
from data.basic.set import Set, subset_contains, subset_contains_eq, compl_contains_eq,
    intersection_contains_eq, intersection_contains_intro, union_contains_eq,
    universal_set_contains_eq, empty_set_contains_eq
from pair import Pair
from analysis.topology.topological_space import TopologicalSpace, big_union,
    big_union_contains_of_member, is_open_cover, is_open_cover_iff, is_compact,
    is_subfamily, is_subfamily_cons_mp, is_subfamily_cons_mpr, list_union_of_sets,
    list_union_of_sets_nil, list_union_of_sets_cons, list_inter_of_sets,
    list_inter_of_sets_nil, list_inter_of_sets_cons, is_hausdorff, hausdorff_inst,
    is_closed, open_inter, open_universal, disjoint_not_both, is_neighborhood,
    open_iff_neighborhood_of_each_point, empty_set_is_compact
from analysis.topology.compact_union import compact_open_cover_has_finite_subcover, compact_union

/// Every member of a finite subfamily belongs to the original family.
theorem subfamily_member_imp_family[T](c: Set[T] -> Bool, items: List[Set[T]], member: Set[T]) {
    is_subfamily[T](c, items) and items.contains(member) implies c(member)
} by {
    define p(xs: List[Set[T]]) -> Bool {
        is_subfamily[T](c, xs) and xs.contains(member) implies c(member)
    }
    if is_subfamily[T](c, List.nil[Set[T]]) and List.nil[Set[T]].contains(member) {
        nil_not_contains[Set[T]](member)
        false
    }
    p(List.nil[Set[T]])
    forall(head: Set[T], tail: List[Set[T]]) {
        if p(tail) {
            if is_subfamily[T](c, List.cons(head, tail)) and List.cons(head, tail).contains(member) {
                is_subfamily_cons_mp[T](c, head, tail)
                if head = member {
                    c(member)
                } else {
                    c(member)
                }
            }
            p(List.cons(head, tail))
        }
        p(tail) implies p(List.cons(head, tail))
    }
    p(List.nil[Set[T]]) and forall(head: Set[T], tail: List[Set[T]]) {
        p(tail) implies p(List.cons(head, tail))
    }
    List.induction(function(xs: List[Set[T]]) { p(xs) })
    forall(xs: List[Set[T]]) { p(xs) }
}

/// Membership in a finite union comes from some member of the list.
theorem list_union_contains_witness[T](items: List[Set[T]], x: T) {
    list_union_of_sets[T](items).contains(x) implies exists(member: Set[T]) {
        items.contains(member) and member.contains(x)
    }
} by {
    define p(xs: List[Set[T]]) -> Bool {
        list_union_of_sets[T](xs).contains(x) implies exists(member: Set[T]) {
            xs.contains(member) and member.contains(x)
        }
    }
    if list_union_of_sets[T](List.nil[Set[T]]).contains(x) {
        list_union_of_sets_nil[T]
        let e: Set[T] = Set[T].empty_set
        forall(y: T) {
            empty_set_contains_eq[T](y)
            not e.contains(y)
        }
        false
    }
    p(List.nil[Set[T]])
    forall(head: Set[T], tail: List[Set[T]]) {
        if p(tail) {
            if not list_union_of_sets[T](List.cons(head, tail)).contains(x) {
                p(List.cons(head, tail))
            }
            if list_union_of_sets[T](List.cons(head, tail)).contains(x) {
                list_union_of_sets_cons[T](head, tail)
                union_contains_eq[T](head, list_union_of_sets[T](tail), x)
                if head.contains(x) {
                    exists(member: Set[T]) {
                        List.cons(head, tail).contains(member) and member.contains(x)
                    }
                } else {
                    list_union_of_sets[T](tail).contains(x)
                    p(tail) = (list_union_of_sets[T](tail).contains(x) implies exists(member: Set[T]) {
                        tail.contains(member) and member.contains(x)
                    })
                    let member: Set[T] satisfy {
                        tail.contains(member) and member.contains(x)
                    }
                    exists(result: Set[T]) {
                        List.cons(head, tail).contains(result) and result.contains(x)
                    }
                }
            }
            p(List.cons(head, tail))
        }
        p(tail) implies p(List.cons(head, tail))
    }
    p(List.nil[Set[T]]) and forall(head: Set[T], tail: List[Set[T]]) {
        p(tail) implies p(List.cons(head, tail))
    }
    List.induction(function(xs: List[Set[T]]) { p(xs) })
    forall(xs: List[Set[T]]) { p(xs) }
}

/// The intersection of the sets in a finite list of open sets is open.
theorem list_inter_of_sets_open[T: TopologicalSpace](items: List[Set[T]]) {
    (forall(u: Set[T]) { items.contains(u) implies T.is_open(u) })
        implies T.is_open(list_inter_of_sets[T](items))
} by {
    define p(xs: List[Set[T]]) -> Bool {
        (forall(u: Set[T]) { xs.contains(u) implies T.is_open(u) })
            implies T.is_open(list_inter_of_sets[T](xs))
    }
    if forall(u: Set[T]) { List.nil[Set[T]].contains(u) implies T.is_open(u) } {
        list_inter_of_sets_nil[T]
        open_universal[T]
        T.is_open(Set[T].universal_set)
        T.is_open(list_inter_of_sets[T](List.nil[Set[T]]))
    }
    p(List.nil[Set[T]])
    forall(head: Set[T], tail: List[Set[T]]) {
        if p(tail) {
            if forall(u: Set[T]) { List.cons(head, tail).contains(u) implies T.is_open(u) } {
                list_inter_of_sets_cons[T](head, tail)
                cons_contains_head[Set[T]](head, tail)
                forall(u: Set[T]) { List.cons(head, tail).contains(u) implies T.is_open(u) }
                T.is_open(head)
                forall(u: Set[T]) {
                    if tail.contains(u) {
                        cons_contains_of_tail_contains[Set[T]](head, tail, u)
                        forall(v: Set[T]) { List.cons(head, tail).contains(v) implies T.is_open(v) }
                        T.is_open(u)
                    }
                }
                p(tail) = ((forall(u: Set[T]) { tail.contains(u) implies T.is_open(u) })
                    implies T.is_open(list_inter_of_sets[T](tail)))
                forall(u: Set[T]) { tail.contains(u) implies T.is_open(u) }
                T.is_open(list_inter_of_sets[T](tail))
                open_inter[T](head, list_inter_of_sets[T](tail))
                T.is_open(head.intersection(list_inter_of_sets[T](tail)))
                T.is_open(list_inter_of_sets[T](List.cons(head, tail)))
            }
            p(List.cons(head, tail))
        }
        p(tail) implies p(List.cons(head, tail))
    }
    p(List.nil[Set[T]]) and forall(head: Set[T], tail: List[Set[T]]) {
        p(tail) implies p(List.cons(head, tail))
    }
    List.induction(function(xs: List[Set[T]]) { p(xs) })
    forall(xs: List[Set[T]]) { p(xs) }
    p(items) = ((forall(u: Set[T]) { items.contains(u) implies T.is_open(u) })
        implies T.is_open(list_inter_of_sets[T](items)))
    if forall(u: Set[T]) { items.contains(u) implies T.is_open(u) } {
        T.is_open(list_inter_of_sets[T](items))
    }
}

/// If every member of a finite list of sets contains `x`, then their intersection contains `x`.
theorem list_inter_of_sets_contains[T](items: List[Set[T]], x: T) {
    (forall(u: Set[T]) { items.contains(u) implies u.contains(x) })
        implies list_inter_of_sets[T](items).contains(x)
} by {
    define p(xs: List[Set[T]]) -> Bool {
        (forall(u: Set[T]) { xs.contains(u) implies u.contains(x) })
            implies list_inter_of_sets[T](xs).contains(x)
    }
    if forall(u: Set[T]) { List.nil[Set[T]].contains(u) implies u.contains(x) } {
        list_inter_of_sets_nil[T]
        universal_set_contains_eq[T](x)
        Set[T].universal_set.contains(x)
        list_inter_of_sets[T](List.nil[Set[T]]).contains(x)
    }
    p(List.nil[Set[T]])
    forall(head: Set[T], tail: List[Set[T]]) {
        if p(tail) {
            if forall(u: Set[T]) { List.cons(head, tail).contains(u) implies u.contains(x) } {
                list_inter_of_sets_cons[T](head, tail)
                cons_contains_head[Set[T]](head, tail)
                forall(u: Set[T]) { List.cons(head, tail).contains(u) implies u.contains(x) }
                head.contains(x)
                forall(u: Set[T]) {
                    if tail.contains(u) {
                        cons_contains_of_tail_contains[Set[T]](head, tail, u)
                        forall(v: Set[T]) { List.cons(head, tail).contains(v) implies v.contains(x) }
                        u.contains(x)
                    }
                }
                p(tail) = ((forall(u: Set[T]) { tail.contains(u) implies u.contains(x) })
                    implies list_inter_of_sets[T](tail).contains(x))
                forall(u: Set[T]) { tail.contains(u) implies u.contains(x) }
                list_inter_of_sets[T](tail).contains(x)
                intersection_contains_intro[T](head, list_inter_of_sets[T](tail), x)
                head.intersection(list_inter_of_sets[T](tail)).contains(x)
                list_inter_of_sets[T](List.cons(head, tail)).contains(x)
            }
            p(List.cons(head, tail))
        }
        p(tail) implies p(List.cons(head, tail))
    }
    p(List.nil[Set[T]]) and forall(head: Set[T], tail: List[Set[T]]) {
        p(tail) implies p(List.cons(head, tail))
    }
    List.induction(function(xs: List[Set[T]]) { p(xs) })
    forall(xs: List[Set[T]]) { p(xs) }
    p(items) = ((forall(u: Set[T]) { items.contains(u) implies u.contains(x) })
        implies list_inter_of_sets[T](items).contains(x))
    if forall(u: Set[T]) { items.contains(u) implies u.contains(x) } {
        list_inter_of_sets[T](items).contains(x)
    }
}

/// Membership in a list intersection gives membership in every member of the list.
theorem list_inter_of_sets_contains_member[T](items: List[Set[T]], u: Set[T], z: T) {
    items.contains(u) and list_inter_of_sets[T](items).contains(z) implies u.contains(z)
} by {
    define p(xs: List[Set[T]]) -> Bool {
        xs.contains(u) and list_inter_of_sets[T](xs).contains(z) implies u.contains(z)
    }
    if List.nil[Set[T]].contains(u) and list_inter_of_sets[T](List.nil[Set[T]]).contains(z) {
        nil_not_contains[Set[T]](u)
        false
    }
    p(List.nil[Set[T]])
    forall(head: Set[T], tail: List[Set[T]]) {
        if p(tail) {
            if List.cons(head, tail).contains(u)
                and list_inter_of_sets[T](List.cons(head, tail)).contains(z) {
                cons_contains_cases[Set[T]](head, tail, u)
                list_inter_of_sets_cons[T](head, tail)
                intersection_contains_eq[T](head, list_inter_of_sets[T](tail), z)
                head.contains(z)
                list_inter_of_sets[T](tail).contains(z)
                if head = u {
                    u.contains(z)
                } else {
                    tail.contains(u)
                    p(tail)
                    u.contains(z)
                }
            }
            p(List.cons(head, tail))
        }
        p(tail) implies p(List.cons(head, tail))
    }
    p(List.nil[Set[T]]) and forall(head: Set[T], tail: List[Set[T]]) {
        p(tail) implies p(List.cons(head, tail))
    }
    List.induction(function(xs: List[Set[T]]) { p(xs) })
    forall(xs: List[Set[T]]) { p(xs) }
    if items.contains(u) and list_inter_of_sets[T](items).contains(z) {
        u.contains(z)
    }
}

/// True if a member `v` of a family is an open neighborhood of some point `y` of `s`
/// that is disjoint from some open neighborhood of `x`: the cover members used in the
/// Hausdorff-separation argument.
define hausdorff_cover_family[T: TopologicalSpace](s: Set[T], x: T, v: Set[T]) -> Bool {
    exists(u: Set[T], y: T) {
        s.contains(y) and T.is_open(u) and T.is_open(v) and u.contains(x) and v.contains(y)
            and u.is_disjoint(v)
    }
}

/// Unfolding lemma for `hausdorff_cover_family`.
theorem hausdorff_cover_family_iff[T: TopologicalSpace](s: Set[T], x: T, v: Set[T]) {
    hausdorff_cover_family(s, x, v) = exists(u: Set[T], y: T) {
        s.contains(y) and T.is_open(u) and T.is_open(v) and u.contains(x) and v.contains(y)
            and u.is_disjoint(v)
    }
} by {
}

/// A chosen open neighborhood of `x` disjoint from a cover member `v`: well-defined
/// whenever `v` lies in the Hausdorff cover family of `s` away from `x`.
let hausdorff_u_choice[T: TopologicalSpace](s: Set[T], x: T, v: Set[T]) -> result: Set[T] satisfy {
    hausdorff_cover_family(s, x, v) implies (T.is_open(result) and result.contains(x)
        and result.is_disjoint(v))
} by {
    if hausdorff_cover_family(s, x, v) {
        hausdorff_cover_family_iff(s, x, v)
        let u: Set[T] satisfy {
            exists(y: T) {
                s.contains(y) and T.is_open(u) and T.is_open(v) and u.contains(x)
                    and v.contains(y) and u.is_disjoint(v)
            }
        }
        let y: T satisfy {
            s.contains(y) and T.is_open(u) and T.is_open(v) and u.contains(x)
                and v.contains(y) and u.is_disjoint(v)
        }
        exists(result: Set[T]) {
            hausdorff_cover_family(s, x, v) implies (T.is_open(result) and result.contains(x)
                and result.is_disjoint(v))
        }
    }
}

/// The chosen neighborhood is open, contains `x`, and is disjoint from the cover member.
theorem hausdorff_u_choice_spec[T: TopologicalSpace](s: Set[T], x: T, v: Set[T]) {
    hausdorff_cover_family(s, x, v) implies (T.is_open(hausdorff_u_choice(s, x, v))
        and hausdorff_u_choice(s, x, v).contains(x)
        and hausdorff_u_choice(s, x, v).is_disjoint(v))
} by {
    if hausdorff_cover_family(s, x, v) {
        hausdorff_u_choice(s, x, v) = hausdorff_u_choice(s, x, v)
        T.is_open(hausdorff_u_choice(s, x, v))
        hausdorff_u_choice(s, x, v).contains(x)
        hausdorff_u_choice(s, x, v).is_disjoint(v)
        (T.is_open(hausdorff_u_choice(s, x, v)) and hausdorff_u_choice(s, x, v).contains(x)
            and hausdorff_u_choice(s, x, v).is_disjoint(v))
    }
}

/// The list of chosen `x`-neighborhoods corresponding to a finite list of cover members.
define hausdorff_u_list[T: TopologicalSpace](s: Set[T], x: T, items: List[Set[T]]) -> List[Set[T]] {
    match items {
        List.nil {
            List.nil[Set[T]]
        }
        List.cons(head, tail) {
            List.cons(hausdorff_u_choice(s, x, head), hausdorff_u_list(s, x, tail))
        }
    }
}

/// The chosen-neighborhood list of an empty cover list is empty.
theorem hausdorff_u_list_nil[T: TopologicalSpace](s: Set[T], x: T) {
    hausdorff_u_list(s, x, List.nil[Set[T]]) = List.nil[Set[T]]
}

/// The chosen-neighborhood list of a cons is the corresponding cons.
theorem hausdorff_u_list_cons[T: TopologicalSpace](s: Set[T], x: T, head: Set[T], tail: List[Set[T]]) {
    hausdorff_u_list(s, x, List.cons(head, tail)) =
        List.cons(hausdorff_u_choice(s, x, head), hausdorff_u_list(s, x, tail))
}

/// A cover member in the list contributes its chosen neighborhood to the chosen list.
theorem hausdorff_u_list_contains_of_items_contains[T: TopologicalSpace](
    s: Set[T], x: T, items: List[Set[T]], v: Set[T]) {
    items.contains(v) implies hausdorff_u_list(s, x, items).contains(hausdorff_u_choice(s, x, v))
} by {
    define p(xs: List[Set[T]]) -> Bool {
        xs.contains(v) implies hausdorff_u_list(s, x, xs).contains(hausdorff_u_choice(s, x, v))
    }
    if List.nil[Set[T]].contains(v) {
        nil_not_contains[Set[T]](v)
        false
    }
    p(List.nil[Set[T]])
    forall(head: Set[T], tail: List[Set[T]]) {
        if p(tail) {
            if List.cons(head, tail).contains(v) {
                cons_contains_cases[Set[T]](head, tail, v)
                if head = v {
                    hausdorff_u_list_cons(s, x, head, tail)
                    cons_contains_head[Set[T]](hausdorff_u_choice(s, x, head),
                        hausdorff_u_list(s, x, tail))
                    hausdorff_u_list(s, x, List.cons(head, tail)).contains(hausdorff_u_choice(s, x, v))
                } else {
                    tail.contains(v)
                    p(tail)
                    hausdorff_u_list(s, x, tail).contains(hausdorff_u_choice(s, x, v))
                    hausdorff_u_list_cons(s, x, head, tail)
                    cons_contains_of_tail_contains[Set[T]](hausdorff_u_choice(s, x, head),
                        hausdorff_u_list(s, x, tail), hausdorff_u_choice(s, x, v))
                    hausdorff_u_list(s, x, List.cons(head, tail)).contains(hausdorff_u_choice(s, x, v))
                }
            }
            p(List.cons(head, tail))
        }
        p(tail) implies p(List.cons(head, tail))
    }
    p(List.nil[Set[T]]) and forall(head: Set[T], tail: List[Set[T]]) {
        p(tail) implies p(List.cons(head, tail))
    }
    List.induction(function(xs: List[Set[T]]) { p(xs) })
    forall(xs: List[Set[T]]) { p(xs) }
    if items.contains(v) {
        hausdorff_u_list(s, x, items).contains(hausdorff_u_choice(s, x, v))
    }
}

/// Every member of the chosen list is open, when the cover list is a subfamily.
theorem hausdorff_u_list_all_open[T: TopologicalSpace](s: Set[T], x: T, items: List[Set[T]]) {
    is_subfamily[T](hausdorff_cover_family(s, x), items) implies
        (forall(u: Set[T]) { hausdorff_u_list(s, x, items).contains(u) implies T.is_open(u) })
} by {
    define p(xs: List[Set[T]]) -> Bool {
        is_subfamily[T](hausdorff_cover_family(s, x), xs) implies
            (forall(u: Set[T]) { hausdorff_u_list(s, x, xs).contains(u) implies T.is_open(u) })
    }
    if is_subfamily[T](hausdorff_cover_family(s, x), List.nil[Set[T]]) {
        hausdorff_u_list_nil(s, x)
        forall(u: Set[T]) {
            if List.nil[Set[T]].contains(u) {
                nil_not_contains[Set[T]](u)
                false
            }
        }
    }
    p(List.nil[Set[T]])
    forall(head: Set[T], tail: List[Set[T]]) {
        if p(tail) {
            if is_subfamily[T](hausdorff_cover_family(s, x), List.cons(head, tail)) {
                is_subfamily_cons_mp[T](hausdorff_cover_family(s, x), head, tail)
                hausdorff_u_list_cons(s, x, head, tail)
                forall(u: Set[T]) {
                    if List.cons(hausdorff_u_choice(s, x, head), hausdorff_u_list(s, x, tail)).contains(u) {
                        cons_contains_cases[Set[T]](hausdorff_u_choice(s, x, head),
                            hausdorff_u_list(s, x, tail), u)
                        if hausdorff_u_choice(s, x, head) = u {
                            hausdorff_u_choice_spec(s, x, head)
                            T.is_open(hausdorff_u_choice(s, x, head))
                            T.is_open(u)
                        } else {
                            p(tail) = (is_subfamily[T](hausdorff_cover_family(s, x), tail) implies
                                (forall(w: Set[T]) { hausdorff_u_list(s, x, tail).contains(w) implies T.is_open(w) }))
                            is_subfamily[T](hausdorff_cover_family(s, x), tail)
                            forall(w: Set[T]) { hausdorff_u_list(s, x, tail).contains(w) implies T.is_open(w) }
                            hausdorff_u_list(s, x, tail).contains(u)
                            T.is_open(u)
                        }
                    }
                }
                forall(u: Set[T]) {
                    hausdorff_u_list(s, x, List.cons(head, tail)).contains(u) implies T.is_open(u)
                }
            }
            p(List.cons(head, tail))
        }
        p(tail) implies p(List.cons(head, tail))
    }
    p(List.nil[Set[T]]) and forall(head: Set[T], tail: List[Set[T]]) {
        p(tail) implies p(List.cons(head, tail))
    }
    List.induction(function(xs: List[Set[T]]) { p(xs) })
    forall(xs: List[Set[T]]) { p(xs) }
    if is_subfamily[T](hausdorff_cover_family(s, x), items) {
        forall(u: Set[T]) { hausdorff_u_list(s, x, items).contains(u) implies T.is_open(u) }
    }
}

/// Every member of the chosen list contains `x`, when the cover list is a subfamily.
theorem hausdorff_u_list_all_contains_x[T: TopologicalSpace](s: Set[T], x: T, items: List[Set[T]]) {
    is_subfamily[T](hausdorff_cover_family(s, x), items) implies
        (forall(u: Set[T]) { hausdorff_u_list(s, x, items).contains(u) implies u.contains(x) })
} by {
    define p(xs: List[Set[T]]) -> Bool {
        is_subfamily[T](hausdorff_cover_family(s, x), xs) implies
            (forall(u: Set[T]) { hausdorff_u_list(s, x, xs).contains(u) implies u.contains(x) })
    }
    if is_subfamily[T](hausdorff_cover_family(s, x), List.nil[Set[T]]) {
        hausdorff_u_list_nil(s, x)
        forall(u: Set[T]) {
            if List.nil[Set[T]].contains(u) {
                nil_not_contains[Set[T]](u)
                false
            }
        }
    }
    p(List.nil[Set[T]])
    forall(head: Set[T], tail: List[Set[T]]) {
        if p(tail) {
            if is_subfamily[T](hausdorff_cover_family(s, x), List.cons(head, tail)) {
                is_subfamily_cons_mp[T](hausdorff_cover_family(s, x), head, tail)
                hausdorff_u_list_cons(s, x, head, tail)
                forall(u: Set[T]) {
                    if List.cons(hausdorff_u_choice(s, x, head), hausdorff_u_list(s, x, tail)).contains(u) {
                        cons_contains_cases[Set[T]](hausdorff_u_choice(s, x, head),
                            hausdorff_u_list(s, x, tail), u)
                        if hausdorff_u_choice(s, x, head) = u {
                            hausdorff_u_choice_spec(s, x, head)
                            hausdorff_u_choice(s, x, head).contains(x)
                            u.contains(x)
                        } else {
                            p(tail) = (is_subfamily[T](hausdorff_cover_family(s, x), tail) implies
                                (forall(w: Set[T]) { hausdorff_u_list(s, x, tail).contains(w) implies w.contains(x) }))
                            is_subfamily[T](hausdorff_cover_family(s, x), tail)
                            forall(w: Set[T]) { hausdorff_u_list(s, x, tail).contains(w) implies w.contains(x) }
                            hausdorff_u_list(s, x, tail).contains(u)
                            u.contains(x)
                        }
                    }
                }
                forall(u: Set[T]) {
                    hausdorff_u_list(s, x, List.cons(head, tail)).contains(u) implies u.contains(x)
                }
            }
            p(List.cons(head, tail))
        }
        p(tail) implies p(List.cons(head, tail))
    }
    p(List.nil[Set[T]]) and forall(head: Set[T], tail: List[Set[T]]) {
        p(tail) implies p(List.cons(head, tail))
    }
    List.induction(function(xs: List[Set[T]]) { p(xs) })
    forall(xs: List[Set[T]]) { p(xs) }
    if is_subfamily[T](hausdorff_cover_family(s, x), items) {
        forall(u: Set[T]) { hausdorff_u_list(s, x, items).contains(u) implies u.contains(x) }
    }
}

/// A compact subset of a Hausdorff space is closed: every point outside the set has an
/// open neighborhood disjoint from it.
theorem compact_imp_closed_in_hausdorff[T: TopologicalSpace](s: Set[T]) {
    is_compact[T](s) and is_hausdorff[T](T.is_open) implies is_closed[T](s)
} by {
    if is_compact[T](s) and is_hausdorff[T](T.is_open) {
        forall(x: T) {
            if s.c.contains(x) {
                compl_contains_eq[T](s, x)
                not s.contains(x)
                let c: Set[T] -> Bool = hausdorff_cover_family(s, x)
                forall(v: Set[T]) {
                    if c(v) {
                        hausdorff_cover_family_iff(s, x, v)
                        let u: Set[T] satisfy {
                            exists(y: T) {
                                s.contains(y) and T.is_open(u) and T.is_open(v) and u.contains(x)
                                    and v.contains(y) and u.is_disjoint(v)
                            }
                        }
                        let y: T satisfy {
                            s.contains(y) and T.is_open(u) and T.is_open(v) and u.contains(x)
                                and v.contains(y) and u.is_disjoint(v)
                        }
                        T.is_open(v)
                    }
                }
                forall(y: T) {
                    if s.contains(y) {
                        if x = y {
                            s.contains(x)
                            not s.contains(x)
                            false
                        }
                        x != y
                        hausdorff_inst[T](T.is_open, x, y)
                        let u: Set[T] satisfy {
                            exists(v: Set[T]) {
                                T.is_open(u) and T.is_open(v) and u.contains(x) and v.contains(y)
                                    and u.is_disjoint(v)
                            }
                        }
                        let v: Set[T] satisfy {
                            T.is_open(u) and T.is_open(v) and u.contains(x) and v.contains(y)
                                and u.is_disjoint(v)
                        }
                        hausdorff_cover_family_iff(s, x, v)
                        c(v)
                        big_union_contains_of_member(c, v, y)
                        big_union(c).contains(y)
                    }
                }
                subset_contains_eq[T](s, big_union(c))
                s.subset(big_union(c))
                is_open_cover_iff[T](c, s)
                is_open_cover[T](c, s)
                compact_open_cover_has_finite_subcover[T](c, s)
                let items: List[Set[T]] satisfy {
                    is_subfamily[T](c, items) and s.subset(list_union_of_sets[T](items))
                }
                let ul: List[Set[T]] = hausdorff_u_list(s, x, items)
                let u_neigh: Set[T] = list_inter_of_sets[T](ul)
                hausdorff_u_list_all_open(s, x, items)
                forall(u: Set[T]) { hausdorff_u_list(s, x, items).contains(u) implies T.is_open(u) }
                list_inter_of_sets_open[T](hausdorff_u_list(s, x, items))
                T.is_open(list_inter_of_sets[T](hausdorff_u_list(s, x, items)))
                hausdorff_u_list_all_contains_x(s, x, items)
                forall(u: Set[T]) { hausdorff_u_list(s, x, items).contains(u) implies u.contains(x) }
                list_inter_of_sets_contains[T](hausdorff_u_list(s, x, items), x)
                list_inter_of_sets[T](hausdorff_u_list(s, x, items)).contains(x)
                forall(z: T) {
                    if u_neigh.contains(z) {
                        if s.contains(z) {
                            subset_contains[T](s, list_union_of_sets[T](items), z)
                            list_union_of_sets[T](items).contains(z)
                            list_union_contains_witness[T](items, z)
                            let v_mem: Set[T] satisfy {
                                items.contains(v_mem) and v_mem.contains(z)
                            }
                            subfamily_member_imp_family[T](c, items, v_mem)
                            c(v_mem)
                            hausdorff_u_choice_spec(s, x, v_mem)
                            hausdorff_u_choice(s, x, v_mem).is_disjoint(v_mem)
                            hausdorff_u_list_contains_of_items_contains(s, x, items, v_mem)
                            ul.contains(hausdorff_u_choice(s, x, v_mem))
                            list_inter_of_sets_contains_member[T](ul, hausdorff_u_choice(s, x, v_mem), z)
                            hausdorff_u_choice(s, x, v_mem).contains(z)
                            disjoint_not_both[T](hausdorff_u_choice(s, x, v_mem), v_mem, z)
                            not (hausdorff_u_choice(s, x, v_mem).contains(z) and v_mem.contains(z))
                            false
                        }
                        not s.contains(z)
                        compl_contains_eq[T](s, z)
                        s.c.contains(z)
                    }
                }
                subset_contains_eq[T](u_neigh, s.c)
                u_neigh.subset(s.c)
                T.is_open(u_neigh) and u_neigh.contains(x) and u_neigh.subset(s.c)
                exists(w: Set[T]) {
                    T.is_open(w) and w.contains(x) and w.subset(s.c)
                }
                is_neighborhood(s.c, x)
            }
        }
        open_iff_neighborhood_of_each_point[T](s.c)
        forall(x: T) {
            s.c.contains(x) implies is_neighborhood(s.c, x)
        }
        T.is_open(s.c)
        is_closed[T](s)
    }
}

/// The union of the sets in a finite list of compact sets is compact.
theorem compact_union_list[T: TopologicalSpace](items: List[Set[T]]) {
    (forall(k: Set[T]) { items.contains(k) implies is_compact[T](k) })
        implies is_compact[T](list_union_of_sets[T](items))
} by {
    define p(xs: List[Set[T]]) -> Bool {
        (forall(k: Set[T]) { xs.contains(k) implies is_compact[T](k) })
            implies is_compact[T](list_union_of_sets[T](xs))
    }
    if forall(k: Set[T]) { List.nil[Set[T]].contains(k) implies is_compact[T](k) } {
        list_union_of_sets_nil[T]
        empty_set_is_compact[T]
        is_compact[T](list_union_of_sets[T](List.nil[Set[T]]))
    }
    p(List.nil[Set[T]])
    forall(head: Set[T], tail: List[Set[T]]) {
        if p(tail) {
            if forall(k: Set[T]) { List.cons(head, tail).contains(k) implies is_compact[T](k) } {
                cons_contains_head[Set[T]](head, tail)
                forall(k: Set[T]) { List.cons(head, tail).contains(k) implies is_compact[T](k) }
                is_compact[T](head)
                forall(k: Set[T]) {
                    if tail.contains(k) {
                        cons_contains_of_tail_contains[Set[T]](head, tail, k)
                        forall(w: Set[T]) { List.cons(head, tail).contains(w) implies is_compact[T](w) }
                        is_compact[T](k)
                    }
                }
                p(tail) = ((forall(k: Set[T]) { tail.contains(k) implies is_compact[T](k) })
                    implies is_compact[T](list_union_of_sets[T](tail)))
                forall(k: Set[T]) { tail.contains(k) implies is_compact[T](k) }
                is_compact[T](list_union_of_sets[T](tail))
                compact_union[T](head, list_union_of_sets[T](tail))
                list_union_of_sets_cons[T](head, tail)
                is_compact[T](list_union_of_sets[T](List.cons(head, tail)))
            }
            p(List.cons(head, tail))
        }
        p(tail) implies p(List.cons(head, tail))
    }
    p(List.nil[Set[T]]) and forall(head: Set[T], tail: List[Set[T]]) {
        p(tail) implies p(List.cons(head, tail))
    }
    List.induction(function(xs: List[Set[T]]) { p(xs) })
    forall(xs: List[Set[T]]) { p(xs) }
    p(items) = ((forall(k: Set[T]) { items.contains(k) implies is_compact[T](k) })
        implies is_compact[T](list_union_of_sets[T](items)))
    if forall(k: Set[T]) { items.contains(k) implies is_compact[T](k) } {
        is_compact[T](list_union_of_sets[T](items))
    }
}
