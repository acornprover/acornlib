from data.basic.set import Set, empty_set_contains_eq, universal_set_contains_eq,
    intersection_contains_eq
from analysis.topology.topological_space import TopologicalSpace, big_union, big_union_contains_eq,
    generated_open, generated_open_empty, generated_open_universal,
    basis_member_is_open, generated_open_inter, generated_open_big_union,
    basis_covers, basis_filters, is_topological_basis

/// A bundled topological basis predicate.
structure GeneratedTopologyBasis[T] {
    /// The family of basic open subsets.
    pred: Set[T] -> Bool
} constraint {
    is_topological_basis(pred)
}

/// The predicate carried by a bundled basis is a topological basis.
theorem generated_topology_basis_is_topological_basis[T](beta: GeneratedTopologyBasis[T]) {
    is_topological_basis(beta.pred)
}

/// A bundled basis covers the whole underlying type.
theorem generated_topology_basis_covers[T](beta: GeneratedTopologyBasis[T]) {
    basis_covers(beta.pred)
} by {
    generated_topology_basis_is_topological_basis[T](beta)
}

/// Carrier wrapper for the topology generated by the bundled basis `beta` on `T`.
structure GeneratedTopology[T, beta: GeneratedTopologyBasis[T]] {
    /// The underlying point of `T`.
    value: T
}

/// True if the generated-topology set `u` is represented by a generated-open set on `T`.
define generated_topology_open[T](
    beta: GeneratedTopologyBasis[T], u: Set[GeneratedTopology[T, beta]]
) -> Bool {
    exists(v: Set[T]) {
        generated_open(beta.pred, v) and forall(p: GeneratedTopology[T, beta]) {
            u.contains(p) = v.contains(p.value)
        }
    }
}

/// The empty set is open in the generated topology wrapper.
theorem generated_topology_open_empty[T](beta: GeneratedTopologyBasis[T]) {
    generated_topology_open[T](beta, Set[GeneratedTopology[T, beta]].empty_set)
} by {
    let e = Set[T].empty_set
    let ee = Set[GeneratedTopology[T, beta]].empty_set
    generated_open_empty[T](beta.pred)
    forall(p: GeneratedTopology[T, beta]) {
        empty_set_contains_eq[GeneratedTopology[T, beta]](p)
        empty_set_contains_eq[T](p.value)
        ee.contains(p) = e.contains(p.value)
    }
    exists(v: Set[T]) {
        generated_open(beta.pred, v) and forall(p: GeneratedTopology[T, beta]) {
            ee.contains(p) = v.contains(p.value)
        }
    }
}

/// The universal set is open in the generated topology wrapper.
theorem generated_topology_open_universal[T](beta: GeneratedTopologyBasis[T]) {
    generated_topology_open[T](beta, Set[GeneratedTopology[T, beta]].universal_set)
} by {
    let u = Set[T].universal_set
    let uu = Set[GeneratedTopology[T, beta]].universal_set
    generated_topology_basis_covers[T](beta)
    generated_open_universal[T](beta.pred)
    generated_open(beta.pred, u)
    forall(p: GeneratedTopology[T, beta]) {
        universal_set_contains_eq[GeneratedTopology[T, beta]](p)
        universal_set_contains_eq[T](p.value)
        uu.contains(p) = u.contains(p.value)
    }
}

/// A trace of a basic member is open in the generated topology wrapper.
theorem generated_topology_basic_open[T](
    beta: GeneratedTopologyBasis[T], s: Set[T], u: Set[GeneratedTopology[T, beta]]
) {
    beta.pred(s) and forall(p: GeneratedTopology[T, beta]) {
        u.contains(p) = s.contains(p.value)
    } implies generated_topology_open[T](beta, u)
} by {
    if beta.pred(s) and forall(p: GeneratedTopology[T, beta]) {
        u.contains(p) = s.contains(p.value)
    } {
        basis_member_is_open[T](beta.pred, s)
        exists(v: Set[T]) {
            generated_open(beta.pred, v) and forall(p: GeneratedTopology[T, beta]) {
                u.contains(p) = v.contains(p.value)
            }
        }
    }
}

/// The wrapper generated-open predicate is closed under binary intersections.
theorem generated_topology_open_inter[T](
    beta: GeneratedTopologyBasis[T], u: Set[GeneratedTopology[T, beta]], w: Set[GeneratedTopology[T, beta]]
) {
    generated_topology_open[T](beta, u) and generated_topology_open[T](beta, w)
        implies generated_topology_open[T](beta, u.intersection(w))
} by {
    if generated_topology_open[T](beta, u) and generated_topology_open[T](beta, w) {
        let v1: Set[T] satisfy {
            generated_open(beta.pred, v1) and forall(p: GeneratedTopology[T, beta]) {
                u.contains(p) = v1.contains(p.value)
            }
        }
        let v2: Set[T] satisfy {
            generated_open(beta.pred, v2) and forall(p: GeneratedTopology[T, beta]) {
                w.contains(p) = v2.contains(p.value)
            }
        }
        let v3 = v1.intersection(v2)
        generated_topology_basis_is_topological_basis[T](beta)
        generated_open_inter[T](beta.pred, v1, v2)
        generated_open(beta.pred, v3)
        forall(p: GeneratedTopology[T, beta]) {
            intersection_contains_eq(u, w, p)
            intersection_contains_eq(v1, v2, p.value)
            u.intersection(w).contains(p) = v3.contains(p.value)
        }
        exists(v: Set[T]) {
            generated_open(beta.pred, v) and forall(p: GeneratedTopology[T, beta]) {
                u.intersection(w).contains(p) = v.contains(p.value)
            }
        }
    }
}

/// Base-space witnesses for a family of generated-topology opens.
define generated_topology_witness_family[T](
    beta: GeneratedTopologyBasis[T], fam: Set[GeneratedTopology[T, beta]] -> Bool, v: Set[T]
) -> Bool {
    generated_open(beta.pred, v) and exists(u: Set[GeneratedTopology[T, beta]]) {
        fam(u) and forall(p: GeneratedTopology[T, beta]) {
            u.contains(p) = v.contains(p.value)
        }
    }
}

/// The wrapper generated-open predicate is closed under arbitrary unions.
theorem generated_topology_open_big_union[T](
    beta: GeneratedTopologyBasis[T], fam: Set[GeneratedTopology[T, beta]] -> Bool
) {
    (forall(u: Set[GeneratedTopology[T, beta]]) {
        fam(u) implies generated_topology_open[T](beta, u)
    }) implies generated_topology_open[T](beta, big_union(fam))
} by {
    if forall(u: Set[GeneratedTopology[T, beta]]) {
        fam(u) implies generated_topology_open[T](beta, u)
    } {
        let q = generated_topology_witness_family[T](beta, fam)
        forall(v: Set[T]) {
            if q(v) {
                generated_open(beta.pred, v)
            }
        }
        generated_open_big_union[T](beta.pred, q)
        let big_v = big_union(q)
        generated_open(beta.pred, big_v)
        forall(p: GeneratedTopology[T, beta]) {
            big_union_contains_eq(fam, p)
            big_union_contains_eq(q, p.value)
            if big_union(fam).contains(p) {
                let u: Set[GeneratedTopology[T, beta]] satisfy {
                    fam(u) and u.contains(p)
                }
                let v: Set[T] satisfy {
                    generated_open(beta.pred, v) and forall(pp: GeneratedTopology[T, beta]) {
                        u.contains(pp) = v.contains(pp.value)
                    }
                }
                exists(w: Set[GeneratedTopology[T, beta]]) {
                    fam(w) and forall(pp: GeneratedTopology[T, beta]) {
                        w.contains(pp) = v.contains(pp.value)
                    }
                }
                q(v)
                big_v.contains(p.value)
            }
            if big_v.contains(p.value) {
                let v: Set[T] satisfy {
                    q(v) and v.contains(p.value)
                }
                let u: Set[GeneratedTopology[T, beta]] satisfy {
                    fam(u) and forall(pp: GeneratedTopology[T, beta]) {
                        u.contains(pp) = v.contains(pp.value)
                    }
                }
                big_union(fam).contains(p)
            }
            big_union(fam).contains(p) = big_v.contains(p.value)
        }
        generated_topology_open[T](beta, big_union(fam))
    }
}

instance GeneratedTopology[T, beta: GeneratedTopologyBasis[T]]: TopologicalSpace {
    let is_open: Set[GeneratedTopology[T, beta]] -> Bool = generated_topology_open[T](beta)
}
