/// Partial bijections between sets, the foundational data for charts on manifolds.

from data.basic.functions import compose, identity_fn, function_extensionality, compose_assoc
from data.basic.set import Set, set_preimage, set_preimage_contains_eq, set_preimage_intersection, set_preimage_compose,
    intersection_contains_eq, intersection_contains_intro, intersection_contains_left, intersection_assoc, set_ext

/// True if `to_fun`, `inv_fun`, `source`, and `target` describe a partial bijection.
/// `to_fun` sends `source` into `target`, `inv_fun` sends `target` into `source`,
/// and the two maps are mutually inverse on their respective sets.
define is_local_equiv_data[A, B](source: Set[A], target: Set[B],
    to_fun: A -> B, inv_fun: B -> A) -> Bool {
    forall(x: A) {
        source.contains(x) implies target.contains(to_fun(x))
    } and forall(y: B) {
        target.contains(y) implies source.contains(inv_fun(y))
    } and forall(x: A) {
        source.contains(x) implies inv_fun(to_fun(x)) = x
    } and forall(y: B) {
        target.contains(y) implies to_fun(inv_fun(y)) = y
    }
}

/// The forward map of a partial bijection sends the source into the target.
theorem local_equiv_data_map_source[A, B](source: Set[A], target: Set[B],
    to_fun: A -> B, inv_fun: B -> A, x: A) {
    is_local_equiv_data(source, target, to_fun, inv_fun) and source.contains(x)
        implies target.contains(to_fun(x))
} by {
    if is_local_equiv_data(source, target, to_fun, inv_fun) and source.contains(x) {
        is_local_equiv_data(source, target, to_fun, inv_fun) = (forall(a: A) {
            source.contains(a) implies target.contains(to_fun(a))
        } and forall(b: B) {
            target.contains(b) implies source.contains(inv_fun(b))
        } and forall(a: A) {
            source.contains(a) implies inv_fun(to_fun(a)) = a
        } and forall(b: B) {
            target.contains(b) implies to_fun(inv_fun(b)) = b
        })
        target.contains(to_fun(x))
    }
}

/// The inverse map of a partial bijection sends the target into the source.
theorem local_equiv_data_map_target[A, B](source: Set[A], target: Set[B],
    to_fun: A -> B, inv_fun: B -> A, y: B) {
    is_local_equiv_data(source, target, to_fun, inv_fun) and target.contains(y)
        implies source.contains(inv_fun(y))
} by {
    if is_local_equiv_data(source, target, to_fun, inv_fun) and target.contains(y) {
        is_local_equiv_data(source, target, to_fun, inv_fun) = (forall(a: A) {
            source.contains(a) implies target.contains(to_fun(a))
        } and forall(b: B) {
            target.contains(b) implies source.contains(inv_fun(b))
        } and forall(a: A) {
            source.contains(a) implies inv_fun(to_fun(a)) = a
        } and forall(b: B) {
            target.contains(b) implies to_fun(inv_fun(b)) = b
        })
        source.contains(inv_fun(y))
    }
}

/// On the source, the inverse undoes the forward map.
theorem local_equiv_data_left_inv[A, B](source: Set[A], target: Set[B],
    to_fun: A -> B, inv_fun: B -> A, x: A) {
    is_local_equiv_data(source, target, to_fun, inv_fun) and source.contains(x)
        implies inv_fun(to_fun(x)) = x
} by {
    if is_local_equiv_data(source, target, to_fun, inv_fun) and source.contains(x) {
        is_local_equiv_data(source, target, to_fun, inv_fun) = (forall(a: A) {
            source.contains(a) implies target.contains(to_fun(a))
        } and forall(b: B) {
            target.contains(b) implies source.contains(inv_fun(b))
        } and forall(a: A) {
            source.contains(a) implies inv_fun(to_fun(a)) = a
        } and forall(b: B) {
            target.contains(b) implies to_fun(inv_fun(b)) = b
        })
        inv_fun(to_fun(x)) = x
    }
}

/// On the target, the forward map undoes the inverse.
theorem local_equiv_data_right_inv[A, B](source: Set[A], target: Set[B],
    to_fun: A -> B, inv_fun: B -> A, y: B) {
    is_local_equiv_data(source, target, to_fun, inv_fun) and target.contains(y)
        implies to_fun(inv_fun(y)) = y
} by {
    if is_local_equiv_data(source, target, to_fun, inv_fun) and target.contains(y) {
        is_local_equiv_data(source, target, to_fun, inv_fun) = (forall(a: A) {
            source.contains(a) implies target.contains(to_fun(a))
        } and forall(b: B) {
            target.contains(b) implies source.contains(inv_fun(b))
        } and forall(a: A) {
            source.contains(a) implies inv_fun(to_fun(a)) = a
        } and forall(b: B) {
            target.contains(b) implies to_fun(inv_fun(b)) = b
        })
        to_fun(inv_fun(y)) = y
    }
}

/// Swapping the roles of source/target and forward/inverse preserves partial bijections.
theorem local_equiv_data_swap[A, B](source: Set[A], target: Set[B],
    to_fun: A -> B, inv_fun: B -> A) {
    is_local_equiv_data(source, target, to_fun, inv_fun)
        implies is_local_equiv_data(target, source, inv_fun, to_fun)
} by {
    if is_local_equiv_data(source, target, to_fun, inv_fun) {
        is_local_equiv_data(source, target, to_fun, inv_fun) = (forall(a: A) {
            source.contains(a) implies target.contains(to_fun(a))
        } and forall(b: B) {
            target.contains(b) implies source.contains(inv_fun(b))
        } and forall(a: A) {
            source.contains(a) implies inv_fun(to_fun(a)) = a
        } and forall(b: B) {
            target.contains(b) implies to_fun(inv_fun(b)) = b
        })
        forall(b: B) {
            target.contains(b) implies source.contains(inv_fun(b))
        }
        forall(a: A) {
            source.contains(a) implies target.contains(to_fun(a))
        }
        forall(b: B) {
            target.contains(b) implies to_fun(inv_fun(b)) = b
        }
        forall(a: A) {
            source.contains(a) implies inv_fun(to_fun(a)) = a
        }
        is_local_equiv_data(target, source, inv_fun, to_fun)
    }
}

/// A partial bijection between sets, with explicit inverse data.
structure LocalEquiv[A, B] {
    /// The set on which the forward map is defined.
    source: Set[A]

    /// The set on which the inverse map is defined.
    target: Set[B]

    /// The forward partial map.
    to_fun: A -> B

    /// The inverse partial map.
    inv_fun: B -> A
} constraint {
    is_local_equiv_data(source, target, to_fun, inv_fun)
}

/// Local equivalences are equal when all four bundled components agree.
theorem local_equiv_ext[A, B](e: LocalEquiv[A, B], e2: LocalEquiv[A, B]) {
    e.source = e2.source and e.target = e2.target and e.to_fun = e2.to_fun and
    e.inv_fun = e2.inv_fun implies e = e2
} by {
    if e.source = e2.source and e.target = e2.target and e.to_fun = e2.to_fun and
        e.inv_fun = e2.inv_fun {
        e.inv_fun = e2.inv_fun
    }
}

/// The forward map of a local equivalence sends its source into its target.
theorem local_equiv_map_source[A, B](e: LocalEquiv[A, B], x: A) {
    e.source.contains(x) implies e.target.contains(e.to_fun(x))
} by {
    if e.source.contains(x) {
        local_equiv_data_map_source(e.source, e.target, e.to_fun, e.inv_fun, x)
        e.target.contains(e.to_fun(x))
    }
}

/// The inverse map of a local equivalence sends its target into its source.
theorem local_equiv_map_target[A, B](e: LocalEquiv[A, B], y: B) {
    e.target.contains(y) implies e.source.contains(e.inv_fun(y))
} by {
    if e.target.contains(y) {
        local_equiv_data_map_target(e.source, e.target, e.to_fun, e.inv_fun, y)
        e.source.contains(e.inv_fun(y))
    }
}

/// On the source, the inverse map undoes the forward map.
theorem local_equiv_left_inv[A, B](e: LocalEquiv[A, B], x: A) {
    e.source.contains(x) implies e.inv_fun(e.to_fun(x)) = x
} by {
    if e.source.contains(x) {
        local_equiv_data_left_inv(e.source, e.target, e.to_fun, e.inv_fun, x)
        e.inv_fun(e.to_fun(x)) = x
    }
}

/// On the target, the forward map undoes the inverse map.
theorem local_equiv_right_inv[A, B](e: LocalEquiv[A, B], y: B) {
    e.target.contains(y) implies e.to_fun(e.inv_fun(y)) = y
} by {
    if e.target.contains(y) {
        local_equiv_data_right_inv(e.source, e.target, e.to_fun, e.inv_fun, y)
        e.to_fun(e.inv_fun(y)) = y
    }
}

/// The source is unchanged after restricting it to points mapped into the target.
theorem local_equiv_source_intersection_preimage_target[A, B](e: LocalEquiv[A, B]) {
    e.source.intersection(set_preimage(e.to_fun, e.target)) = e.source
} by {
    let u = e.source.intersection(set_preimage(e.to_fun, e.target))
    let v = e.source
    forall(x: A) {
        if u.contains(x) {
            intersection_contains_left(e.source, set_preimage(e.to_fun, e.target), x)
            v.contains(x)
        }
        if v.contains(x) {
            local_equiv_data_map_source(e.source, e.target, e.to_fun, e.inv_fun, x)
            e.target.contains(e.to_fun(x))
            set_preimage_contains_eq(e.to_fun, e.target, x)
            set_preimage(e.to_fun, e.target).contains(x)
            intersection_contains_intro(e.source, set_preimage(e.to_fun, e.target), x)
            u.contains(x)
        }
        u.contains(x) = v.contains(x)
    }
    set_ext(u, v)
}

/// The target is unchanged after restricting it to points whose inverse maps into the source.
theorem local_equiv_target_intersection_preimage_source[A, B](e: LocalEquiv[A, B]) {
    e.target.intersection(set_preimage(e.inv_fun, e.source)) = e.target
} by {
    let u = e.target.intersection(set_preimage(e.inv_fun, e.source))
    let v = e.target
    forall(y: B) {
        if u.contains(y) {
            intersection_contains_left(e.target, set_preimage(e.inv_fun, e.source), y)
            v.contains(y)
        }
        if v.contains(y) {
            local_equiv_data_map_target(e.source, e.target, e.to_fun, e.inv_fun, y)
            e.source.contains(e.inv_fun(y))
            set_preimage_contains_eq(e.inv_fun, e.source, y)
            set_preimage(e.inv_fun, e.source).contains(y)
            intersection_contains_intro(e.target, set_preimage(e.inv_fun, e.source), y)
            u.contains(y)
        }
        u.contains(y) = v.contains(y)
    }
    set_ext(u, v)
}

/// The identity map on a set defines a partial bijection from the set to itself.
theorem local_equiv_data_id[A](s: Set[A]) {
    is_local_equiv_data(s, s, identity_fn[A], identity_fn[A])
} by {
    forall(x: A) {
        if s.contains(x) {
            s.contains(identity_fn[A](x))
        }
    }
    let p1: Bool = forall(x: A) {
        s.contains(x) implies s.contains(identity_fn[A](x))
    }
    forall(x: A) {
        if s.contains(x) {
            identity_fn[A](identity_fn[A](x)) = x
        }
    }
    let p2: Bool = forall(x: A) {
        s.contains(x) implies identity_fn[A](identity_fn[A](x)) = x
    }
    p2
    is_local_equiv_data(s, s, identity_fn[A], identity_fn[A]) = (p1 and p1 and p2 and p2)
}

/// The identity partial bijection on a set is constructible.
theorem local_equiv_id_constructible[A](source: Set[A]) {
    exists(h: LocalEquiv[A, A]) {
        LocalEquiv[A, A].new(source, source, identity_fn[A], identity_fn[A]) = Option.some(h)
    }
} by {
    local_equiv_data_id(source)
}

/// Restricting a partial bijection's source to a subset still gives a partial bijection,
/// when the target is correspondingly restricted to the inverse preimage.
theorem local_equiv_data_restr[A, B](source: Set[A], target: Set[B],
    to_fun: A -> B, inv_fun: B -> A, s: Set[A]) {
    is_local_equiv_data(source, target, to_fun, inv_fun)
        implies is_local_equiv_data(source.intersection(s),
            target.intersection(set_preimage(inv_fun, s)), to_fun, inv_fun)
} by {
    if is_local_equiv_data(source, target, to_fun, inv_fun) {
        forall(x: A) {
            if source.intersection(s).contains(x) {
                intersection_contains_eq(source, s, x)
                local_equiv_data_map_source(source, target, to_fun, inv_fun, x)
                local_equiv_data_left_inv(source, target, to_fun, inv_fun, x)
                inv_fun(to_fun(x)) = x
                set_preimage_contains_eq(inv_fun, s, to_fun(x))
                set_preimage(inv_fun, s).contains(to_fun(x))
                intersection_contains_intro(target, set_preimage(inv_fun, s), to_fun(x))
                target.intersection(set_preimage(inv_fun, s)).contains(to_fun(x))
            }
        }
        let q1: Bool = forall(a: A) {
            source.intersection(s).contains(a)
                implies target.intersection(set_preimage(inv_fun, s)).contains(to_fun(a))
        }
        forall(y: B) {
            if target.intersection(set_preimage(inv_fun, s)).contains(y) {
                intersection_contains_eq(target, set_preimage(inv_fun, s), y)
                set_preimage_contains_eq(inv_fun, s, y)
                local_equiv_data_map_target(source, target, to_fun, inv_fun, y)
                intersection_contains_intro(source, s, inv_fun(y))
                source.intersection(s).contains(inv_fun(y))
            }
        }
        let q2: Bool = forall(b: B) {
            target.intersection(set_preimage(inv_fun, s)).contains(b)
                implies source.intersection(s).contains(inv_fun(b))
        }
        q2
        forall(x: A) {
            if source.intersection(s).contains(x) {
                intersection_contains_left(source, s, x)
                local_equiv_data_left_inv(source, target, to_fun, inv_fun, x)
                inv_fun(to_fun(x)) = x
            }
        }
        let q3: Bool = forall(a: A) {
            source.intersection(s).contains(a) implies inv_fun(to_fun(a)) = a
        }
        q3
        forall(y: B) {
            if target.intersection(set_preimage(inv_fun, s)).contains(y) {
                intersection_contains_left(target, set_preimage(inv_fun, s), y)
                local_equiv_data_right_inv(source, target, to_fun, inv_fun, y)
                to_fun(inv_fun(y)) = y
            }
        }
        let q4: Bool = forall(b: B) {
            target.intersection(set_preimage(inv_fun, s)).contains(b)
                implies to_fun(inv_fun(b)) = b
        }
        q4
        q1 and q2 and q3 and q4
        is_local_equiv_data(source.intersection(s),
            target.intersection(set_preimage(inv_fun, s)), to_fun, inv_fun)
    }
}

/// The restriction of a partial bijection to a subset of its source is constructible.
theorem local_equiv_restr_constructible[A, B](e: LocalEquiv[A, B], s: Set[A]) {
    exists(h: LocalEquiv[A, B]) {
        LocalEquiv[A, B].new(e.source.intersection(s),
            e.target.intersection(set_preimage(e.inv_fun, s)), e.to_fun, e.inv_fun) = Option.some(h)
    }
} by {
    local_equiv_data_restr(e.source, e.target, e.to_fun, e.inv_fun, s)
}

/// Composing two partial bijections gives a partial bijection on the appropriate restriction.
theorem local_equiv_data_trans[A, B, C](
    s1: Set[A], t1: Set[B], f1: A -> B, g1: B -> A,
    s2: Set[B], t2: Set[C], f2: B -> C, g2: C -> B) {
    is_local_equiv_data(s1, t1, f1, g1) and is_local_equiv_data(s2, t2, f2, g2)
        implies is_local_equiv_data(s1.intersection(set_preimage(f1, s2)),
            t2.intersection(set_preimage(g2, t1)), compose(f2, f1), compose(g1, g2))
} by {
    if is_local_equiv_data(s1, t1, f1, g1) and is_local_equiv_data(s2, t2, f2, g2) {
        forall(x: A) {
            if s1.intersection(set_preimage(f1, s2)).contains(x) {
                intersection_contains_eq(s1, set_preimage(f1, s2), x)
                set_preimage_contains_eq(f1, s2, x)
                s2.contains(f1(x))
                local_equiv_data_map_source(s1, t1, f1, g1, x)
                local_equiv_data_map_source(s2, t2, f2, g2, f1(x))
                local_equiv_data_left_inv(s2, t2, f2, g2, f1(x))
                g2(f2(f1(x))) = f1(x)
                t1.contains(g2(f2(f1(x))))
                set_preimage_contains_eq(g2, t1, f2(f1(x)))
                set_preimage(g2, t1).contains(f2(f1(x)))
                intersection_contains_intro(t2, set_preimage(g2, t1), f2(f1(x)))
                t2.intersection(set_preimage(g2, t1)).contains(f2(f1(x)))
                t2.intersection(set_preimage(g2, t1)).contains(compose(f2, f1)(x))
            }
        }
        let q1: Bool = forall(a: A) {
            s1.intersection(set_preimage(f1, s2)).contains(a)
                implies t2.intersection(set_preimage(g2, t1)).contains(compose(f2, f1)(a))
        }
        forall(y: C) {
            if t2.intersection(set_preimage(g2, t1)).contains(y) {
                intersection_contains_eq(t2, set_preimage(g2, t1), y)
                set_preimage_contains_eq(g2, t1, y)
                t1.contains(g2(y))
                local_equiv_data_map_target(s2, t2, f2, g2, y)
                local_equiv_data_map_target(s1, t1, f1, g1, g2(y))
                local_equiv_data_right_inv(s1, t1, f1, g1, g2(y))
                f1(g1(g2(y))) = g2(y)
                s2.contains(f1(g1(g2(y))))
                set_preimage_contains_eq(f1, s2, g1(g2(y)))
                set_preimage(f1, s2).contains(g1(g2(y)))
                intersection_contains_intro(s1, set_preimage(f1, s2), g1(g2(y)))
                s1.intersection(set_preimage(f1, s2)).contains(g1(g2(y)))
                s1.intersection(set_preimage(f1, s2)).contains(compose(g1, g2)(y))
            }
        }
        let q2: Bool = forall(b: C) {
            t2.intersection(set_preimage(g2, t1)).contains(b)
                implies s1.intersection(set_preimage(f1, s2)).contains(compose(g1, g2)(b))
        }
        q2
        forall(x: A) {
            if s1.intersection(set_preimage(f1, s2)).contains(x) {
                intersection_contains_left(s1, set_preimage(f1, s2), x)
                intersection_contains_eq(s1, set_preimage(f1, s2), x)
                set_preimage_contains_eq(f1, s2, x)
                s2.contains(f1(x))
                local_equiv_data_left_inv(s2, t2, f2, g2, f1(x))
                g2(f2(f1(x))) = f1(x)
                local_equiv_data_left_inv(s1, t1, f1, g1, x)
                g1(g2(f2(f1(x)))) = x
                compose(g1, g2)(compose(f2, f1)(x)) = x
            }
        }
        let q3: Bool = forall(a: A) {
            s1.intersection(set_preimage(f1, s2)).contains(a)
                implies compose(g1, g2)(compose(f2, f1)(a)) = a
        }
        q3
        forall(y: C) {
            if t2.intersection(set_preimage(g2, t1)).contains(y) {
                intersection_contains_left(t2, set_preimage(g2, t1), y)
                intersection_contains_eq(t2, set_preimage(g2, t1), y)
                set_preimage_contains_eq(g2, t1, y)
                t1.contains(g2(y))
                local_equiv_data_right_inv(s1, t1, f1, g1, g2(y))
                f1(g1(g2(y))) = g2(y)
                local_equiv_data_right_inv(s2, t2, f2, g2, y)
                f2(f1(g1(g2(y)))) = y
                compose(f2, f1)(compose(g1, g2)(y)) = y
            }
        }
        let q4: Bool = forall(b: C) {
            t2.intersection(set_preimage(g2, t1)).contains(b)
                implies compose(f2, f1)(compose(g1, g2)(b)) = b
        }
        q4
        q1 and q2 and q3 and q4
        is_local_equiv_data(s1.intersection(set_preimage(f1, s2)),
            t2.intersection(set_preimage(g2, t1)), compose(f2, f1), compose(g1, g2))
    }
}

/// The composition of two partial bijections is constructible.
theorem local_equiv_trans_constructible[A, B, C](e: LocalEquiv[A, B], e2: LocalEquiv[B, C]) {
    exists(h: LocalEquiv[A, C]) {
        LocalEquiv[A, C].new(e.source.intersection(set_preimage(e.to_fun, e2.source)),
            e2.target.intersection(set_preimage(e2.inv_fun, e.target)),
            compose(e2.to_fun, e.to_fun), compose(e.inv_fun, e2.inv_fun)) = Option.some(h)
    }
} by {
    local_equiv_data_trans(e.source, e.target, e.to_fun, e.inv_fun,
        e2.source, e2.target, e2.to_fun, e2.inv_fun)
    is_local_equiv_data(e.source.intersection(set_preimage(e.to_fun, e2.source)),
        e2.target.intersection(set_preimage(e2.inv_fun, e.target)),
        compose(e2.to_fun, e.to_fun), compose(e.inv_fun, e2.inv_fun))
}

/// The swapped data of a partial bijection always defines a partial bijection.
theorem local_equiv_swap_constructible[A, B](e: LocalEquiv[A, B]) {
    exists(h: LocalEquiv[B, A]) {
        LocalEquiv[B, A].new(e.target, e.source, e.inv_fun, e.to_fun) = Option.some(h)
    }
} by {
    local_equiv_data_swap(e.source, e.target, e.to_fun, e.inv_fun)
}

/// The identity local equivalence on a set.
let local_equiv_id[A](source: Set[A]) -> result: LocalEquiv[A, A] satisfy {
    LocalEquiv[A, A].new(source, source, identity_fn[A], identity_fn[A]) = Option.some(result)
} by {
    local_equiv_id_constructible(source)
}

/// The source of the identity local equivalence is its defining set.
theorem local_equiv_id_source[A](source: Set[A]) {
    local_equiv_id(source).source = source
} by {
    LocalEquiv[A, A].new(source, source, identity_fn[A], identity_fn[A]) = Option.some(local_equiv_id(source))
}

/// The target of the identity local equivalence is its defining set.
theorem local_equiv_id_target[A](source: Set[A]) {
    local_equiv_id(source).target = source
} by {
    LocalEquiv[A, A].new(source, source, identity_fn[A], identity_fn[A]) = Option.some(local_equiv_id(source))
}

/// The forward map of the identity local equivalence is the identity function.
theorem local_equiv_id_to_fun[A](source: Set[A]) {
    local_equiv_id(source).to_fun = identity_fn[A]
} by {
    forall(x: A) {
        LocalEquiv[A, A].new(source, source, identity_fn[A], identity_fn[A]) = Option.some(local_equiv_id(source))
        local_equiv_id(source).to_fun(x) = identity_fn[A](x)
    }
    function_extensionality(local_equiv_id(source).to_fun, identity_fn[A])
}

/// The inverse map of the identity local equivalence is the identity function.
theorem local_equiv_id_inv_fun[A](source: Set[A]) {
    local_equiv_id(source).inv_fun = identity_fn[A]
} by {
    forall(x: A) {
        LocalEquiv[A, A].new(source, source, identity_fn[A], identity_fn[A]) = Option.some(local_equiv_id(source))
        local_equiv_id(source).inv_fun(x) = identity_fn[A](x)
    }
    function_extensionality(local_equiv_id(source).inv_fun, identity_fn[A])
}

/// The identity local equivalence fixes every point of its source.
theorem local_equiv_id_apply[A](source: Set[A], x: A) {
    source.contains(x) implies local_equiv_id(source).to_fun(x) = x
} by {
    if source.contains(x) {
        local_equiv_id_to_fun(source)
        local_equiv_id(source).to_fun(x) = x
    }
}

/// The inverse map of the identity local equivalence fixes every point of its target.
theorem local_equiv_id_inv_apply[A](source: Set[A], x: A) {
    source.contains(x) implies local_equiv_id(source).inv_fun(x) = x
} by {
    if source.contains(x) {
        local_equiv_id_inv_fun(source)
        local_equiv_id(source).inv_fun(x) = x
    }
}

/// The restriction of a local equivalence to a subset of its source.
let local_equiv_restr[A, B](e: LocalEquiv[A, B], s: Set[A]) -> result: LocalEquiv[A, B] satisfy {
    LocalEquiv[A, B].new(e.source.intersection(s),
        e.target.intersection(set_preimage(e.inv_fun, s)), e.to_fun, e.inv_fun) = Option.some(result)
} by {
    local_equiv_restr_constructible(e, s)
}

/// The source of a restricted local equivalence is the restricted source.
theorem local_equiv_restr_source[A, B](e: LocalEquiv[A, B], s: Set[A]) {
    local_equiv_restr(e, s).source = e.source.intersection(s)
} by {
    LocalEquiv[A, B].new(e.source.intersection(s),
        e.target.intersection(set_preimage(e.inv_fun, s)), e.to_fun, e.inv_fun) = Option.some(local_equiv_restr(e, s))
}

/// The target of a restricted local equivalence is the corresponding inverse preimage restriction.
theorem local_equiv_restr_target[A, B](e: LocalEquiv[A, B], s: Set[A]) {
    local_equiv_restr(e, s).target = e.target.intersection(set_preimage(e.inv_fun, s))
} by {
    LocalEquiv[A, B].new(e.source.intersection(s),
        e.target.intersection(set_preimage(e.inv_fun, s)), e.to_fun, e.inv_fun) = Option.some(local_equiv_restr(e, s))
}

/// The forward map of a restricted local equivalence is unchanged.
theorem local_equiv_restr_to_fun[A, B](e: LocalEquiv[A, B], s: Set[A]) {
    local_equiv_restr(e, s).to_fun = e.to_fun
} by {
    forall(x: A) {
        LocalEquiv[A, B].new(e.source.intersection(s),
            e.target.intersection(set_preimage(e.inv_fun, s)), e.to_fun, e.inv_fun) = Option.some(local_equiv_restr(e, s))
        local_equiv_restr(e, s).to_fun(x) = e.to_fun(x)
    }
    function_extensionality(local_equiv_restr(e, s).to_fun, e.to_fun)
}

/// The inverse map of a restricted local equivalence is unchanged.
theorem local_equiv_restr_inv_fun[A, B](e: LocalEquiv[A, B], s: Set[A]) {
    local_equiv_restr(e, s).inv_fun = e.inv_fun
} by {
    forall(y: B) {
        LocalEquiv[A, B].new(e.source.intersection(s),
            e.target.intersection(set_preimage(e.inv_fun, s)), e.to_fun, e.inv_fun) = Option.some(local_equiv_restr(e, s))
        local_equiv_restr(e, s).inv_fun(y) = e.inv_fun(y)
    }
    function_extensionality(local_equiv_restr(e, s).inv_fun, e.inv_fun)
}

/// A point in the original source and restricting set lies in the restricted source.
theorem local_equiv_restr_source_contains[A, B](e: LocalEquiv[A, B], s: Set[A], x: A) {
    e.source.contains(x) and s.contains(x) implies local_equiv_restr(e, s).source.contains(x)
} by {
    if e.source.contains(x) and s.contains(x) {
        local_equiv_restr_source(e, s)
        intersection_contains_eq(e.source, s, x)
        local_equiv_restr(e, s).source.contains(x)
    }
}

/// A target point whose inverse lies in the restricting set lies in the restricted target.
theorem local_equiv_restr_target_contains[A, B](e: LocalEquiv[A, B], s: Set[A], y: B) {
    e.target.contains(y) and s.contains(e.inv_fun(y)) implies local_equiv_restr(e, s).target.contains(y)
} by {
    if e.target.contains(y) and s.contains(e.inv_fun(y)) {
        local_equiv_restr_target(e, s)
        set_preimage_contains_eq(e.inv_fun, s, y)
        intersection_contains_eq(e.target, set_preimage(e.inv_fun, s), y)
        local_equiv_restr(e, s).target.contains(y)
    }
}

/// Restriction preserves forward target membership for points that remain in the source.
theorem local_equiv_restr_map_source[A, B](e: LocalEquiv[A, B], s: Set[A], x: A) {
    e.source.contains(x) and s.contains(x) implies local_equiv_restr(e, s).target.contains(e.to_fun(x))
} by {
    if e.source.contains(x) and s.contains(x) {
        local_equiv_map_source(e, x)
        local_equiv_left_inv(e, x)
        local_equiv_restr_target_contains(e, s, e.to_fun(x))
        local_equiv_restr(e, s).target.contains(e.to_fun(x))
    }
}

/// Restriction preserves inverse-source membership for target points that remain in the restricted target.
theorem local_equiv_restr_map_target[A, B](e: LocalEquiv[A, B], s: Set[A], y: B) {
    e.target.contains(y) and s.contains(e.inv_fun(y)) implies
        local_equiv_restr(e, s).source.contains(e.inv_fun(y))
} by {
    if e.target.contains(y) and s.contains(e.inv_fun(y)) {
        local_equiv_map_target(e, y)
        local_equiv_restr_source_contains(e, s, e.inv_fun(y))
        local_equiv_restr(e, s).source.contains(e.inv_fun(y))
    }
}

/// Restricting twice is the same as restricting by the intersection of the two subsets.
theorem local_equiv_restr_restr[A, B](e: LocalEquiv[A, B], s: Set[A], t: Set[A]) {
    local_equiv_restr(local_equiv_restr(e, s), t) = local_equiv_restr(e, s.intersection(t))
} by {
    let lhs = local_equiv_restr(local_equiv_restr(e, s), t)
    let rhs = local_equiv_restr(e, s.intersection(t))
    local_equiv_restr_source(local_equiv_restr(e, s), t)
    local_equiv_restr_source(e, s)
    local_equiv_restr_source(e, s.intersection(t))
    intersection_assoc(e.source, s, t)
    lhs.source = rhs.source
    local_equiv_restr_target(local_equiv_restr(e, s), t)
    local_equiv_restr_target(e, s)
    local_equiv_restr_inv_fun(e, s)
    local_equiv_restr_target(e, s.intersection(t))
    set_preimage_intersection(e.inv_fun, s, t)
    intersection_assoc(e.target, set_preimage(e.inv_fun, s), set_preimage(e.inv_fun, t))
    lhs.target = rhs.target
    local_equiv_restr_to_fun(local_equiv_restr(e, s), t)
    local_equiv_restr_to_fun(e, s)
    local_equiv_restr_to_fun(e, s.intersection(t))
    lhs.to_fun = rhs.to_fun
    local_equiv_restr_inv_fun(local_equiv_restr(e, s), t)
    local_equiv_restr_inv_fun(e, s)
    local_equiv_restr_inv_fun(e, s.intersection(t))
    lhs.inv_fun = rhs.inv_fun
    local_equiv_ext(lhs, rhs)
}

/// The symmetry of a local equivalence, with source and target swapped.
let local_equiv_symm[A, B](e: LocalEquiv[A, B]) -> result: LocalEquiv[B, A] satisfy {
    LocalEquiv[B, A].new(e.target, e.source, e.inv_fun, e.to_fun) = Option.some(result)
} by {
    local_equiv_swap_constructible(e)
}

/// The source of the symmetric local equivalence is the original target.
theorem local_equiv_symm_source[A, B](e: LocalEquiv[A, B]) {
    local_equiv_symm(e).source = e.target
} by {
    LocalEquiv[B, A].new(e.target, e.source, e.inv_fun, e.to_fun) = Option.some(local_equiv_symm(e))
}

/// The target of the symmetric local equivalence is the original source.
theorem local_equiv_symm_target[A, B](e: LocalEquiv[A, B]) {
    local_equiv_symm(e).target = e.source
} by {
    LocalEquiv[B, A].new(e.target, e.source, e.inv_fun, e.to_fun) = Option.some(local_equiv_symm(e))
}

/// The forward map of the symmetric local equivalence is the original inverse map.
theorem local_equiv_symm_to_fun[A, B](e: LocalEquiv[A, B]) {
    local_equiv_symm(e).to_fun = e.inv_fun
} by {
    forall(y: B) {
        LocalEquiv[B, A].new(e.target, e.source, e.inv_fun, e.to_fun) = Option.some(local_equiv_symm(e))
        local_equiv_symm(e).to_fun(y) = e.inv_fun(y)
    }
    function_extensionality(local_equiv_symm(e).to_fun, e.inv_fun)
}

/// The inverse map of the symmetric local equivalence is the original forward map.
theorem local_equiv_symm_inv_fun[A, B](e: LocalEquiv[A, B]) {
    local_equiv_symm(e).inv_fun = e.to_fun
} by {
    forall(x: A) {
        LocalEquiv[B, A].new(e.target, e.source, e.inv_fun, e.to_fun) = Option.some(local_equiv_symm(e))
        local_equiv_symm(e).inv_fun(x) = e.to_fun(x)
    }
    function_extensionality(local_equiv_symm(e).inv_fun, e.to_fun)
}

/// The symmetric local equivalence has the expected left inverse on the original target.
theorem local_equiv_symm_left_inv[A, B](e: LocalEquiv[A, B], y: B) {
    e.target.contains(y) implies
    local_equiv_symm(e).inv_fun(local_equiv_symm(e).to_fun(y)) = y
} by {
    if e.target.contains(y) {
        let h = local_equiv_symm(e)
        local_equiv_symm_source(e)
        is_local_equiv_data(h.source, h.target, h.to_fun, h.inv_fun)
        local_equiv_data_left_inv(h.source, h.target, h.to_fun, h.inv_fun, y)
    }
}

/// The symmetric local equivalence has the expected right inverse on the original source.
theorem local_equiv_symm_right_inv[A, B](e: LocalEquiv[A, B], x: A) {
    e.source.contains(x) implies
    local_equiv_symm(e).to_fun(local_equiv_symm(e).inv_fun(x)) = x
} by {
    if e.source.contains(x) {
        let h = local_equiv_symm(e)
        local_equiv_symm_target(e)
        is_local_equiv_data(h.source, h.target, h.to_fun, h.inv_fun)
        local_equiv_data_right_inv(h.source, h.target, h.to_fun, h.inv_fun, x)
    }
}

/// Symmetry is involutive as a bundled local equivalence.
theorem local_equiv_symm_symm[A, B](e: LocalEquiv[A, B]) {
    local_equiv_symm(local_equiv_symm(e)) = e
} by {
    let lhs = local_equiv_symm(local_equiv_symm(e))
    local_equiv_symm_source(local_equiv_symm(e))
    local_equiv_symm_target(e)
    lhs.source = e.source
    local_equiv_symm_target(local_equiv_symm(e))
    local_equiv_symm_source(e)
    lhs.target = e.target
    local_equiv_symm_to_fun(local_equiv_symm(e))
    local_equiv_symm_inv_fun(e)
    lhs.to_fun = e.to_fun
    local_equiv_symm_inv_fun(local_equiv_symm(e))
    local_equiv_symm_to_fun(e)
    lhs.inv_fun = e.inv_fun
    local_equiv_ext(lhs, e)
}

/// The composition of local equivalences, restricted to the natural source and target.
let local_equiv_trans[A, B, C](e: LocalEquiv[A, B], e2: LocalEquiv[B, C]) -> result: LocalEquiv[A, C] satisfy {
    LocalEquiv[A, C].new(e.source.intersection(set_preimage(e.to_fun, e2.source)),
        e2.target.intersection(set_preimage(e2.inv_fun, e.target)),
        compose(e2.to_fun, e.to_fun), compose(e.inv_fun, e2.inv_fun)) = Option.some(result)
} by {
    local_equiv_trans_constructible(e, e2)
}

/// The source of a composition is the source restricted to where the first map lands in the second source.
theorem local_equiv_trans_source[A, B, C](e: LocalEquiv[A, B], e2: LocalEquiv[B, C]) {
    local_equiv_trans(e, e2).source = e.source.intersection(set_preimage(e.to_fun, e2.source))
} by {
    LocalEquiv[A, C].new(e.source.intersection(set_preimage(e.to_fun, e2.source)),
        e2.target.intersection(set_preimage(e2.inv_fun, e.target)),
        compose(e2.to_fun, e.to_fun), compose(e.inv_fun, e2.inv_fun)) = Option.some(local_equiv_trans(e, e2))
}

/// The target of a composition is the target restricted to where the second inverse lands in the first target.
theorem local_equiv_trans_target[A, B, C](e: LocalEquiv[A, B], e2: LocalEquiv[B, C]) {
    local_equiv_trans(e, e2).target = e2.target.intersection(set_preimage(e2.inv_fun, e.target))
} by {
    LocalEquiv[A, C].new(e.source.intersection(set_preimage(e.to_fun, e2.source)),
        e2.target.intersection(set_preimage(e2.inv_fun, e.target)),
        compose(e2.to_fun, e.to_fun), compose(e.inv_fun, e2.inv_fun)) = Option.some(local_equiv_trans(e, e2))
}

/// The forward map of a composition is the composition of the forward maps.
theorem local_equiv_trans_to_fun[A, B, C](e: LocalEquiv[A, B], e2: LocalEquiv[B, C]) {
    local_equiv_trans(e, e2).to_fun = compose(e2.to_fun, e.to_fun)
} by {
    forall(x: A) {
        LocalEquiv[A, C].new(e.source.intersection(set_preimage(e.to_fun, e2.source)),
            e2.target.intersection(set_preimage(e2.inv_fun, e.target)),
            compose(e2.to_fun, e.to_fun), compose(e.inv_fun, e2.inv_fun)) = Option.some(local_equiv_trans(e, e2))
        local_equiv_trans(e, e2).to_fun(x) = compose(e2.to_fun, e.to_fun, x)
    }
    function_extensionality(local_equiv_trans(e, e2).to_fun, compose(e2.to_fun, e.to_fun))
}

/// The inverse map of a composition is the composition of the inverse maps in reverse order.
theorem local_equiv_trans_inv_fun[A, B, C](e: LocalEquiv[A, B], e2: LocalEquiv[B, C]) {
    local_equiv_trans(e, e2).inv_fun = compose(e.inv_fun, e2.inv_fun)
} by {
    forall(z: C) {
        LocalEquiv[A, C].new(e.source.intersection(set_preimage(e.to_fun, e2.source)),
            e2.target.intersection(set_preimage(e2.inv_fun, e.target)),
            compose(e2.to_fun, e.to_fun), compose(e.inv_fun, e2.inv_fun)) = Option.some(local_equiv_trans(e, e2))
        local_equiv_trans(e, e2).inv_fun(z) = compose(e.inv_fun, e2.inv_fun, z)
    }
    function_extensionality(local_equiv_trans(e, e2).inv_fun, compose(e.inv_fun, e2.inv_fun))
}

/// A point lies in the composition source when it starts in the first source and maps into the second source.
theorem local_equiv_trans_source_contains[A, B, C](e: LocalEquiv[A, B], e2: LocalEquiv[B, C], x: A) {
    e.source.contains(x) and e2.source.contains(e.to_fun(x)) implies local_equiv_trans(e, e2).source.contains(x)
} by {
    if e.source.contains(x) and e2.source.contains(e.to_fun(x)) {
        local_equiv_trans_source(e, e2)
        set_preimage_contains_eq(e.to_fun, e2.source, x)
        intersection_contains_eq(e.source, set_preimage(e.to_fun, e2.source), x)
        local_equiv_trans(e, e2).source.contains(x)
    }
}

/// A point lies in the composition target when its second inverse lands in the first target.
theorem local_equiv_trans_target_contains[A, B, C](e: LocalEquiv[A, B], e2: LocalEquiv[B, C], z: C) {
    e2.target.contains(z) and e.target.contains(e2.inv_fun(z)) implies local_equiv_trans(e, e2).target.contains(z)
} by {
    if e2.target.contains(z) and e.target.contains(e2.inv_fun(z)) {
        local_equiv_trans_target(e, e2)
        set_preimage_contains_eq(e2.inv_fun, e.target, z)
        intersection_contains_eq(e2.target, set_preimage(e2.inv_fun, e.target), z)
        local_equiv_trans(e, e2).target.contains(z)
    }
}

/// Composition maps a compatible source point to the second forward image.
theorem local_equiv_trans_map_source[A, B, C](e: LocalEquiv[A, B], e2: LocalEquiv[B, C], x: A) {
    e.source.contains(x) and e2.source.contains(e.to_fun(x)) implies
        local_equiv_trans(e, e2).target.contains(e2.to_fun(e.to_fun(x)))
} by {
    if e.source.contains(x) and e2.source.contains(e.to_fun(x)) {
        local_equiv_map_source(e, x)
        local_equiv_map_source(e2, e.to_fun(x))
        local_equiv_left_inv(e2, e.to_fun(x))
        local_equiv_trans_target_contains(e, e2, e2.to_fun(e.to_fun(x)))
        local_equiv_trans(e, e2).target.contains(e2.to_fun(e.to_fun(x)))
    }
}

/// Composition maps a compatible target point back to the first inverse image.
theorem local_equiv_trans_map_target[A, B, C](e: LocalEquiv[A, B], e2: LocalEquiv[B, C], z: C) {
    e2.target.contains(z) and e.target.contains(e2.inv_fun(z)) implies
        local_equiv_trans(e, e2).source.contains(e.inv_fun(e2.inv_fun(z)))
} by {
    if e2.target.contains(z) and e.target.contains(e2.inv_fun(z)) {
        local_equiv_map_target(e2, z)
        local_equiv_map_target(e, e2.inv_fun(z))
        local_equiv_right_inv(e, e2.inv_fun(z))
        local_equiv_trans_source_contains(e, e2, e.inv_fun(e2.inv_fun(z)))
        local_equiv_trans(e, e2).source.contains(e.inv_fun(e2.inv_fun(z)))
    }
}

/// On a compatible source point, the inverse of the composition undoes the forward image.
theorem local_equiv_trans_left_inv_apply[A, B, C](e: LocalEquiv[A, B], e2: LocalEquiv[B, C], x: A) {
    e.source.contains(x) and e2.source.contains(e.to_fun(x)) implies
        local_equiv_trans(e, e2).inv_fun(e2.to_fun(e.to_fun(x))) = x
} by {
    if e.source.contains(x) and e2.source.contains(e.to_fun(x)) {
        local_equiv_trans_inv_fun(e, e2)
        local_equiv_left_inv(e2, e.to_fun(x))
        local_equiv_left_inv(e, x)
        e.inv_fun(e2.inv_fun(e2.to_fun(e.to_fun(x)))) = x
        local_equiv_trans(e, e2).inv_fun(e2.to_fun(e.to_fun(x))) = x
    }
}

/// On a compatible target point, the forward map of the composition undoes the inverse image.
theorem local_equiv_trans_right_inv_apply[A, B, C](e: LocalEquiv[A, B], e2: LocalEquiv[B, C], z: C) {
    e2.target.contains(z) and e.target.contains(e2.inv_fun(z)) implies
        local_equiv_trans(e, e2).to_fun(e.inv_fun(e2.inv_fun(z))) = z
} by {
    if e2.target.contains(z) and e.target.contains(e2.inv_fun(z)) {
        local_equiv_trans_to_fun(e, e2)
        local_equiv_right_inv(e, e2.inv_fun(z))
        local_equiv_right_inv(e2, z)
        e2.to_fun(e.to_fun(e.inv_fun(e2.inv_fun(z)))) = z
        local_equiv_trans(e, e2).to_fun(e.inv_fun(e2.inv_fun(z))) = z
    }
}

/// Composition of local equivalences is associative as a bundled local equivalence.
theorem local_equiv_trans_assoc[A, B, C, D](
    e1: LocalEquiv[A, B], e2: LocalEquiv[B, C], e3: LocalEquiv[C, D]) {
    local_equiv_trans(local_equiv_trans(e1, e2), e3) =
        local_equiv_trans(e1, local_equiv_trans(e2, e3))
} by {
    let lhs = local_equiv_trans(local_equiv_trans(e1, e2), e3)
    let rhs = local_equiv_trans(e1, local_equiv_trans(e2, e3))
    local_equiv_trans_source(local_equiv_trans(e1, e2), e3)
    local_equiv_trans_source(e1, e2)
    local_equiv_trans_to_fun(e1, e2)
    local_equiv_trans_source(e2, e3)
    local_equiv_trans_source(e1, local_equiv_trans(e2, e3))
    set_preimage_intersection(e1.to_fun, e2.source, set_preimage(e2.to_fun, e3.source))
    set_preimage_compose(e2.to_fun, e1.to_fun, e3.source)
    intersection_assoc(e1.source, set_preimage(e1.to_fun, e2.source),
        set_preimage(e1.to_fun, set_preimage(e2.to_fun, e3.source)))
    lhs.source = rhs.source
    local_equiv_trans_target(local_equiv_trans(e1, e2), e3)
    local_equiv_trans_target(e1, e2)
    local_equiv_trans_inv_fun(e1, e2)
    local_equiv_trans_target(e2, e3)
    local_equiv_trans_inv_fun(e2, e3)
    local_equiv_trans_target(e1, local_equiv_trans(e2, e3))
    set_preimage_intersection(e3.inv_fun, e2.target, set_preimage(e2.inv_fun, e1.target))
    set_preimage_compose(e2.inv_fun, e3.inv_fun, e1.target)
    intersection_assoc(e3.target, set_preimage(e3.inv_fun, e2.target),
        set_preimage(e3.inv_fun, set_preimage(e2.inv_fun, e1.target)))
    lhs.target = rhs.target
    local_equiv_trans_to_fun(local_equiv_trans(e1, e2), e3)
    local_equiv_trans_to_fun(e1, e2)
    local_equiv_trans_to_fun(e2, e3)
    local_equiv_trans_to_fun(e1, local_equiv_trans(e2, e3))
    compose_assoc(e3.to_fun, e2.to_fun, e1.to_fun)
    lhs.to_fun = rhs.to_fun
    local_equiv_trans_inv_fun(local_equiv_trans(e1, e2), e3)
    local_equiv_trans_inv_fun(e1, e2)
    local_equiv_trans_inv_fun(e2, e3)
    local_equiv_trans_inv_fun(e1, local_equiv_trans(e2, e3))
    compose_assoc(e1.inv_fun, e2.inv_fun, e3.inv_fun)
    lhs.inv_fun = rhs.inv_fun
    local_equiv_ext(lhs, rhs)
}

/// Composing a local equivalence on the left with the identity on its target leaves the local equivalence unchanged.
theorem local_equiv_trans_id_left[A, B](e: LocalEquiv[A, B]) {
    local_equiv_trans(e, local_equiv_id(e.target)) = e
} by {
    let lhs = local_equiv_trans(e, local_equiv_id(e.target))
    local_equiv_trans_source(e, local_equiv_id(e.target))
    local_equiv_id_source(e.target)
    local_equiv_source_intersection_preimage_target(e)
    lhs.source = e.source
    local_equiv_trans_target(e, local_equiv_id(e.target))
    local_equiv_id_target(e.target)
    local_equiv_id_inv_fun(e.target)
    local_equiv_target_intersection_preimage_source(e)
    lhs.target = e.target
    local_equiv_trans_to_fun(e, local_equiv_id(e.target))
    local_equiv_id_to_fun(e.target)
    forall(x: A) {
        compose(local_equiv_id(e.target).to_fun, e.to_fun, x) = e.to_fun(x)
    }
    function_extensionality(compose(local_equiv_id(e.target).to_fun, e.to_fun), e.to_fun)
    lhs.to_fun = e.to_fun
    local_equiv_trans_inv_fun(e, local_equiv_id(e.target))
    local_equiv_id_inv_fun(e.target)
    forall(y: B) {
        compose(e.inv_fun, local_equiv_id(e.target).inv_fun, y) = e.inv_fun(y)
    }
    function_extensionality(compose(e.inv_fun, local_equiv_id(e.target).inv_fun), e.inv_fun)
    lhs.inv_fun = e.inv_fun
    local_equiv_ext(lhs, e)
}

/// Composing a local equivalence on the right with the identity on its source leaves the local equivalence unchanged.
theorem local_equiv_trans_id_right[A, B](e: LocalEquiv[A, B]) {
    local_equiv_trans(local_equiv_id(e.source), e) = e
} by {
    let lhs = local_equiv_trans(local_equiv_id(e.source), e)
    local_equiv_trans_source(local_equiv_id(e.source), e)
    local_equiv_id_source(e.source)
    local_equiv_id_to_fun(e.source)
    local_equiv_source_intersection_preimage_target(e)
    lhs.source = e.source
    local_equiv_trans_target(local_equiv_id(e.source), e)
    local_equiv_id_target(e.source)
    local_equiv_target_intersection_preimage_source(e)
    lhs.target = e.target
    local_equiv_trans_to_fun(local_equiv_id(e.source), e)
    local_equiv_id_to_fun(e.source)
    forall(x: A) {
        compose(e.to_fun, local_equiv_id(e.source).to_fun, x) = e.to_fun(x)
    }
    function_extensionality(compose(e.to_fun, local_equiv_id(e.source).to_fun), e.to_fun)
    lhs.to_fun = e.to_fun
    local_equiv_trans_inv_fun(local_equiv_id(e.source), e)
    local_equiv_id_inv_fun(e.source)
    forall(y: B) {
        compose(local_equiv_id(e.source).inv_fun, e.inv_fun, y) = e.inv_fun(y)
    }
    function_extensionality(compose(local_equiv_id(e.source).inv_fun, e.inv_fun), e.inv_fun)
    lhs.inv_fun = e.inv_fun
    local_equiv_ext(lhs, e)
}

/// The right symmetry composition has source equal to the original source.
theorem local_equiv_trans_symm_right_source[A, B](e: LocalEquiv[A, B]) {
    local_equiv_trans(e, local_equiv_symm(e)).source = e.source
} by {
    local_equiv_trans_source(e, local_equiv_symm(e))
    local_equiv_symm_source(e)
    local_equiv_source_intersection_preimage_target(e)
}

/// The right symmetry composition has target equal to the original source.
theorem local_equiv_trans_symm_right_target[A, B](e: LocalEquiv[A, B]) {
    local_equiv_trans(e, local_equiv_symm(e)).target = e.source
} by {
    local_equiv_trans_target(e, local_equiv_symm(e))
    local_equiv_symm_target(e)
    local_equiv_symm_inv_fun(e)
    local_equiv_source_intersection_preimage_target(e)
}

/// On the original source, composing a local equivalence with its symmetry applies as the identity.
theorem local_equiv_trans_symm_right_apply[A, B](e: LocalEquiv[A, B], x: A) {
    e.source.contains(x) implies local_equiv_trans(e, local_equiv_symm(e)).to_fun(x) = x
} by {
    if e.source.contains(x) {
        local_equiv_trans_to_fun(e, local_equiv_symm(e))
        local_equiv_symm_to_fun(e)
        local_equiv_data_left_inv(e.source, e.target, e.to_fun, e.inv_fun, x)
        e.inv_fun(e.to_fun(x)) = x
        local_equiv_trans(e, local_equiv_symm(e)).to_fun(x) = x
    }
}

/// On the original source, the inverse of the right symmetry composition also applies as the identity.
theorem local_equiv_trans_symm_right_inv_apply[A, B](e: LocalEquiv[A, B], x: A) {
    e.source.contains(x) implies local_equiv_trans(e, local_equiv_symm(e)).inv_fun(x) = x
} by {
    if e.source.contains(x) {
        local_equiv_trans_inv_fun(e, local_equiv_symm(e))
        local_equiv_symm_inv_fun(e)
        local_equiv_data_left_inv(e.source, e.target, e.to_fun, e.inv_fun, x)
        e.inv_fun(e.to_fun(x)) = x
        local_equiv_trans(e, local_equiv_symm(e)).inv_fun(x) = x
    }
}

/// The left symmetry composition has source equal to the original target.
theorem local_equiv_trans_symm_left_source[A, B](e: LocalEquiv[A, B]) {
    local_equiv_trans(local_equiv_symm(e), e).source = e.target
} by {
    local_equiv_trans_source(local_equiv_symm(e), e)
    local_equiv_symm_source(e)
    local_equiv_symm_to_fun(e)
    local_equiv_target_intersection_preimage_source(e)
}

/// The left symmetry composition has target equal to the original target.
theorem local_equiv_trans_symm_left_target[A, B](e: LocalEquiv[A, B]) {
    local_equiv_trans(local_equiv_symm(e), e).target = e.target
} by {
    local_equiv_trans_target(local_equiv_symm(e), e)
    local_equiv_symm_target(e)
    local_equiv_target_intersection_preimage_source(e)
}

/// On the original target, composing the symmetry with the original local equivalence applies as the identity.
theorem local_equiv_trans_symm_left_apply[A, B](e: LocalEquiv[A, B], y: B) {
    e.target.contains(y) implies local_equiv_trans(local_equiv_symm(e), e).to_fun(y) = y
} by {
    if e.target.contains(y) {
        local_equiv_trans_to_fun(local_equiv_symm(e), e)
        local_equiv_symm_to_fun(e)
        local_equiv_data_right_inv(e.source, e.target, e.to_fun, e.inv_fun, y)
        e.to_fun(e.inv_fun(y)) = y
        local_equiv_trans(local_equiv_symm(e), e).to_fun(y) = y
    }
}

/// On the original target, the inverse of the left symmetry composition also applies as the identity.
theorem local_equiv_trans_symm_left_inv_apply[A, B](e: LocalEquiv[A, B], y: B) {
    e.target.contains(y) implies local_equiv_trans(local_equiv_symm(e), e).inv_fun(y) = y
} by {
    if e.target.contains(y) {
        local_equiv_trans_inv_fun(local_equiv_symm(e), e)
        local_equiv_symm_inv_fun(e)
        local_equiv_data_right_inv(e.source, e.target, e.to_fun, e.inv_fun, y)
        e.to_fun(e.inv_fun(y)) = y
        local_equiv_trans(local_equiv_symm(e), e).inv_fun(y) = y
    }
}
