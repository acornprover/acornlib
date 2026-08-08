/// Carrier-side bridges for ambient subspace topology predicates.

from data.basic.set import Set, set_preimage, set_preimage_compl, set_ext, intersection_contains_eq,
    compl_of_compl_is_self, set_eq_transport_predicate
from analysis.topology.topological_space import TopologicalSpace, Subspace, has_subspace_witness,
    subspace_open, is_open_in_subspace, is_closed_in_subspace, is_closed,
    subspace_open_relative_complement_closed, subspace_relative_complement_trace
from analysis.topology.subspace_inclusion import subspace_inclusion, subspace_inclusion_preimage_contains

/// The carrier-side set of subspace points whose ambient values lie in `u`.
define subspace_carrier_set[T: TopologicalSpace](a: Set[T], u: Set[T]) -> Set[Subspace[T, a]] {
    set_preimage(subspace_inclusion[T](a), u)
}

/// Membership in the carrier-side bridge is membership of the ambient value.
theorem subspace_carrier_set_contains[T: TopologicalSpace](
    a: Set[T], u: Set[T], p: Subspace[T, a]
) {
    subspace_carrier_set(a, u).contains(p) = u.contains(p.value)
} by {
    subspace_carrier_set(a, u) = set_preimage(subspace_inclusion[T](a), u)
    subspace_inclusion_preimage_contains(a, u, p)
}

/// Tracing an ambient set to the carrier does not change its carrier-side set.
theorem subspace_carrier_set_trace_eq[T: TopologicalSpace](a: Set[T], u: Set[T], v: Set[T]) {
    u = v.intersection(a) implies subspace_carrier_set(a, u) = subspace_carrier_set(a, v)
} by {
    if u = v.intersection(a) {
        let lhs = subspace_carrier_set(a, u)
        let rhs = subspace_carrier_set(a, v)
        forall(p: Subspace[T, a]) {
            subspace_carrier_set_contains(a, u, p)
            lhs.contains(p) = u.contains(p.value)
            u.contains(p.value) = v.intersection(a).contains(p.value)
            intersection_contains_eq(v, a, p.value)
            v.intersection(a).contains(p.value) = (v.contains(p.value) and a.contains(p.value))
            a.contains(p.value)
            (v.contains(p.value) and a.contains(p.value)) = v.contains(p.value)
            v.intersection(a).contains(p.value) = v.contains(p.value)
            subspace_carrier_set_contains(a, v, p)
            rhs.contains(p) = v.contains(p.value)
            lhs.contains(p) = rhs.contains(p)
        }
        set_ext(lhs, rhs)
        lhs = rhs
        subspace_carrier_set(a, u) = subspace_carrier_set(a, v)
    }
}

/// Complement commutes with moving an ambient set to the subspace carrier.
theorem subspace_carrier_set_compl_eq[T: TopologicalSpace](a: Set[T], u: Set[T]) {
    subspace_carrier_set(a, u).c = subspace_carrier_set(a, u.c)
} by {
    let f = subspace_inclusion[T](a)
    subspace_carrier_set(a, u) = set_preimage(f, u)
    subspace_carrier_set(a, u.c) = set_preimage(f, u.c)
    set_preimage_compl(f, u)
    set_preimage(f, u.c) = set_preimage(f, u).c
    subspace_carrier_set(a, u.c) = subspace_carrier_set(a, u).c
    subspace_carrier_set(a, u).c = subspace_carrier_set(a, u.c)
}

/// The carrier-side preimage of an ambient open set is open in the bundled subspace topology.
theorem subspace_carrier_set_open_from_ambient[T: TopologicalSpace](a: Set[T], v: Set[T]) {
    T.is_open(v) implies Subspace[T, a].is_open(subspace_carrier_set(a, v))
} by {
    if T.is_open(v) {
        let f = subspace_inclusion[T](a)
        let pre = set_preimage(f, v)
        forall(p: Subspace[T, a]) {
            subspace_inclusion_preimage_contains(a, v, p)
            pre.contains(p) = v.contains(p.value)
        }
        T.is_open(v) and forall(p: Subspace[T, a]) {
            pre.contains(p) = v.contains(p.value)
        }
        has_subspace_witness(a, pre, v)
        subspace_open(a, pre)
        Subspace[T, a].is_open(pre)
        subspace_carrier_set(a, v) = pre
        Subspace[T, a].is_open(subspace_carrier_set(a, v))
    }
}

/// A predicate-side subspace-open ambient set gives an open set in the bundled subspace carrier.
theorem subspace_carrier_set_open_of_open_in_subspace[T: TopologicalSpace](a: Set[T], u: Set[T]) {
    is_open_in_subspace[T](a, u) implies Subspace[T, a].is_open(subspace_carrier_set(a, u))
} by {
    if is_open_in_subspace[T](a, u) {
        let v: Set[T] satisfy {
            T.is_open(v) and u = v.intersection(a)
        }
        subspace_carrier_set_trace_eq(a, u, v)
        subspace_carrier_set(a, u) = subspace_carrier_set(a, v)
        subspace_carrier_set_open_from_ambient(a, v)
        Subspace[T, a].is_open(subspace_carrier_set(a, v))
        Subspace[T, a].is_open(subspace_carrier_set(a, u))
    }
}

/// An explicit carrier-side ambient witness recovers a predicate-side subspace-open set.
theorem subspace_open_in_subspace_of_carrier_witness[T: TopologicalSpace](a: Set[T], u: Set[T], v: Set[T]) {
    T.is_open(v) and
        (forall(p: Subspace[T, a]) { subspace_carrier_set(a, u).contains(p) = v.contains(p.value) })
        implies is_open_in_subspace[T](a, u.intersection(a))
} by {
    if T.is_open(v) and
        (forall(p: Subspace[T, a]) { subspace_carrier_set(a, u).contains(p) = v.contains(p.value) }) {
        let lhs = u.intersection(a)
        let rhs = v.intersection(a)
        forall(x: T) {
            intersection_contains_eq(u, a, x)
            intersection_contains_eq(v, a, x)
            if lhs.contains(x) {
                u.contains(x)
                a.contains(x)
                let p: Subspace[T, a] satisfy {
                    Subspace[T, a].new(x) = Option.some(p)
                }
                p.value = x
                subspace_carrier_set_contains(a, u, p)
                subspace_carrier_set(a, u).contains(p) = u.contains(p.value)
                u.contains(p.value)
                subspace_carrier_set(a, u).contains(p)
                subspace_carrier_set(a, u).contains(p) = v.contains(p.value)
                v.contains(p.value)
                v.contains(x)
                rhs.contains(x)
            }
            if rhs.contains(x) {
                v.contains(x)
                a.contains(x)
                let p: Subspace[T, a] satisfy {
                    Subspace[T, a].new(x) = Option.some(p)
                }
                p.value = x
                subspace_carrier_set(a, u).contains(p) = v.contains(p.value)
                v.contains(p.value)
                subspace_carrier_set(a, u).contains(p)
                subspace_carrier_set_contains(a, u, p)
                subspace_carrier_set(a, u).contains(p) = u.contains(p.value)
                u.contains(p.value)
                u.contains(x)
                lhs.contains(x)
            }
            lhs.contains(x) = rhs.contains(x)
        }
        set_ext(lhs, rhs)
        lhs = rhs
        T.is_open(v) and u.intersection(a) = v.intersection(a)
        exists(w: Set[T]) {
            T.is_open(w) and u.intersection(a) = w.intersection(a)
        }
        is_open_in_subspace[T](a, u.intersection(a))
    }
}

/// If the carrier-side bridge is open in the bundled subspace topology, then the
/// ambient trace is open in the predicate-side subspace API.
theorem subspace_open_in_subspace_of_carrier_open[T: TopologicalSpace](a: Set[T], u: Set[T]) {
    Subspace[T, a].is_open(subspace_carrier_set(a, u))
        implies is_open_in_subspace[T](a, u.intersection(a))
} by {
    if Subspace[T, a].is_open(subspace_carrier_set(a, u)) {
        subspace_open(a, subspace_carrier_set(a, u))
        let v: Set[T] satisfy {
            has_subspace_witness(a, subspace_carrier_set(a, u), v)
        }
        has_subspace_witness(a, subspace_carrier_set(a, u), v) =
            (T.is_open(v) and forall(p: Subspace[T, a]) {
                subspace_carrier_set(a, u).contains(p) = v.contains(p.value)
            })
        T.is_open(v)
        forall(p: Subspace[T, a]) {
            subspace_carrier_set(a, u).contains(p) = v.contains(p.value)
        }
        subspace_open_in_subspace_of_carrier_witness[T](a, u, v)
        is_open_in_subspace[T](a, u.intersection(a))
    }
}

/// The bundled carrier-side and predicate-side subspace openness APIs agree on
/// the trace of an ambient set.
theorem subspace_carrier_open_iff[T: TopologicalSpace](a: Set[T], u: Set[T]) {
    Subspace[T, a].is_open(subspace_carrier_set(a, u)) =
        is_open_in_subspace[T](a, u.intersection(a))
} by {
    if Subspace[T, a].is_open(subspace_carrier_set(a, u)) {
        subspace_open_in_subspace_of_carrier_open[T](a, u)
        is_open_in_subspace[T](a, u.intersection(a))
    }
    if is_open_in_subspace[T](a, u.intersection(a)) {
        subspace_carrier_set_open_of_open_in_subspace[T](a, u.intersection(a))
        Subspace[T, a].is_open(subspace_carrier_set(a, u.intersection(a)))
        subspace_carrier_set_trace_eq[T](a, u.intersection(a), u)
        subspace_carrier_set(a, u.intersection(a)) = subspace_carrier_set(a, u)
        let p: Set[Subspace[T, a]] -> Bool = function(s: Set[Subspace[T, a]]) {
            Subspace[T, a].is_open(s)
        }
        set_eq_transport_predicate[Subspace[T, a]](p,
            subspace_carrier_set(a, u.intersection(a)), subspace_carrier_set(a, u))
        Subspace[T, a].is_open(subspace_carrier_set(a, u))
    }
}

/// The carrier-side preimage of an ambient closed set is closed in the bundled subspace topology.
theorem subspace_carrier_set_closed_from_ambient[T: TopologicalSpace](a: Set[T], c: Set[T]) {
    is_closed[T](c) implies is_closed[Subspace[T, a]](subspace_carrier_set(a, c))
} by {
    if is_closed[T](c) {
        is_closed[T](c) = T.is_open(c.c)
        T.is_open(c.c)
        subspace_carrier_set_open_from_ambient(a, c.c)
        Subspace[T, a].is_open(subspace_carrier_set(a, c.c))
        subspace_carrier_set_compl_eq(a, c)
        subspace_carrier_set(a, c).c = subspace_carrier_set(a, c.c)
        is_closed[Subspace[T, a]](subspace_carrier_set(a, c)) =
            Subspace[T, a].is_open(subspace_carrier_set(a, c).c)
        Subspace[T, a].is_open(subspace_carrier_set(a, c).c)
        is_closed[Subspace[T, a]](subspace_carrier_set(a, c))
    }
}

/// A predicate-side subspace-closed ambient set gives a closed set in the bundled subspace carrier.
theorem subspace_carrier_set_closed_of_closed_in_subspace[T: TopologicalSpace](a: Set[T], u: Set[T]) {
    is_closed_in_subspace[T](a, u) implies is_closed[Subspace[T, a]](subspace_carrier_set(a, u))
} by {
    if is_closed_in_subspace[T](a, u) {
        let c: Set[T] satisfy {
            is_closed[T](c) and u = c.intersection(a)
        }
        subspace_carrier_set_trace_eq(a, u, c)
        subspace_carrier_set(a, u) = subspace_carrier_set(a, c)
        subspace_carrier_set_closed_from_ambient(a, c)
        is_closed[Subspace[T, a]](subspace_carrier_set(a, c))
        is_closed[Subspace[T, a]](subspace_carrier_set(a, u))
    }
}

/// If the carrier-side bridge is closed in the bundled subspace topology, then the
/// ambient trace is closed in the predicate-side subspace API.
theorem subspace_closed_in_subspace_of_carrier_closed[T: TopologicalSpace](a: Set[T], u: Set[T]) {
    is_closed[Subspace[T, a]](subspace_carrier_set(a, u))
        implies is_closed_in_subspace[T](a, u.intersection(a))
} by {
    if is_closed[Subspace[T, a]](subspace_carrier_set(a, u)) {
        is_closed[Subspace[T, a]](subspace_carrier_set(a, u)) =
            Subspace[T, a].is_open(subspace_carrier_set(a, u).c)
        Subspace[T, a].is_open(subspace_carrier_set(a, u).c)
        subspace_carrier_set_compl_eq[T](a, u)
        subspace_carrier_set(a, u).c = subspace_carrier_set(a, u.c)
        let p: Set[Subspace[T, a]] -> Bool = function(s: Set[Subspace[T, a]]) {
            Subspace[T, a].is_open(s)
        }
        set_eq_transport_predicate[Subspace[T, a]](p,
            subspace_carrier_set(a, u).c, subspace_carrier_set(a, u.c))
        Subspace[T, a].is_open(subspace_carrier_set(a, u.c))
        subspace_open_in_subspace_of_carrier_open[T](a, u.c)
        is_open_in_subspace[T](a, u.c.intersection(a))
        subspace_open_relative_complement_closed[T](a, u.c.intersection(a))
        is_closed_in_subspace[T](a, a.difference(u.c.intersection(a)))
        subspace_relative_complement_trace[T](a, u.c)
        a.difference(u.c.intersection(a)) = u.c.c.intersection(a)
        compl_of_compl_is_self[T](u)
        u.c.c = u
        u.c.c.intersection(a) = u.intersection(a)
        a.difference(u.c.intersection(a)) = u.intersection(a)
        let q: Set[T] -> Bool = function(s: Set[T]) {
            is_closed_in_subspace[T](a, s)
        }
        set_eq_transport_predicate[T](q, a.difference(u.c.intersection(a)), u.intersection(a))
        is_closed_in_subspace[T](a, u.intersection(a))
    }
}

/// The bundled carrier-side and predicate-side subspace closedness APIs agree on
/// the trace of an ambient set.
theorem subspace_carrier_closed_iff[T: TopologicalSpace](a: Set[T], u: Set[T]) {
    is_closed[Subspace[T, a]](subspace_carrier_set(a, u)) =
        is_closed_in_subspace[T](a, u.intersection(a))
} by {
    if is_closed[Subspace[T, a]](subspace_carrier_set(a, u)) {
        subspace_closed_in_subspace_of_carrier_closed[T](a, u)
        is_closed_in_subspace[T](a, u.intersection(a))
    }
    if is_closed_in_subspace[T](a, u.intersection(a)) {
        subspace_carrier_set_closed_of_closed_in_subspace[T](a, u.intersection(a))
        is_closed[Subspace[T, a]](subspace_carrier_set(a, u.intersection(a)))
        subspace_carrier_set_trace_eq[T](a, u.intersection(a), u)
        subspace_carrier_set(a, u.intersection(a)) = subspace_carrier_set(a, u)
        let p: Set[Subspace[T, a]] -> Bool = function(s: Set[Subspace[T, a]]) {
            is_closed[Subspace[T, a]](s)
        }
        set_eq_transport_predicate[Subspace[T, a]](p,
            subspace_carrier_set(a, u.intersection(a)), subspace_carrier_set(a, u))
        is_closed[Subspace[T, a]](subspace_carrier_set(a, u))
    }
}
