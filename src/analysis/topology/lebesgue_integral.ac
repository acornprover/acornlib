/// The Lebesgue integral of simple functions.
///
/// A simple function on the real line is a finite linear combination of
/// indicator functions of Lebesgue measurable sets.  It is represented here
/// by a finite index set `values`, a family `sets` assigning to each index a
/// measurable set, and a family `coeffs` assigning to each index its real
/// coefficient; the function is
///     f(x) = sum_{a in values} coeffs(a) * 1_{sets(a)}(x),
/// and its integral is
///     int f = sum_{a in values} coeffs(a) * lambda(sets(a)),
/// where `lambda` is the Lebesgue outer measure.  Since the outer measure is
/// only meaningful (equal to the infimum of the cover costs) when the cost set
/// has an infimum, the theorems asserting inequalities for integrals carry the
/// hypothesis `lebesgue_cost_has_infimum(sets(a))` for each index.
from data.basic.functions import function_extensionality
from data.basic.set import Set, list_set, list_set_contains_eq
from nat import Nat
from list import List, map, sum, map_singleton, singleton_contains_imp_eq, singleton_unique
from real import Real, mul_nonneg, mul_distrib_left, mul_one_left, mul_assoc
from algebra.add_semigroup import add_fn
from algebra.semigroup import mul_fn
from order import lt_imp_lte
from ordered_field import zero_is_smaller_than_one
from finite_set import FiniteSet, fs_from_list, finite_set_sum, finite_set_sum_add,
    finite_set_sum_scalar_mul, finite_set_sum_eq_list_sum
from data.finite.finite_set_sum_real import finite_set_sum_nonneg
from data.finite.finite_set_sum_product_extra import finite_set_sum_eq_of_pointwise_on_set
from analysis.topology.lebesgue import lebesgue_outer_measure, lebesgue_measurable,
    lebesgue_cost_has_infimum, lebesgue_outer_measure_nonneg

numerals Real

/// The real-valued indicator function of a set of reals: one on the set and
/// zero off it.
define real_indicator(s: Set[Real], x: Real) -> Real {
    if s.contains(x) { Real.1 } else { Real.0 }
}

/// The indicator function of a set is one on the set.
theorem real_indicator_one_of_contains(s: Set[Real], x: Real) {
    s.contains(x) implies real_indicator(s, x) = Real.1
} by {
    if s.contains(x) {
        real_indicator(s, x) = Real.1
    }
}

/// The indicator function of a set is zero off the set.
theorem real_indicator_zero_of_not_contains(s: Set[Real], x: Real) {
    not s.contains(x) implies real_indicator(s, x) = Real.0
} by {
    if not s.contains(x) {
        real_indicator(s, x) = Real.0
    }
}

/// The indicator function of a set is nonnegative.
theorem real_indicator_nonneg(s: Set[Real], x: Real) {
    Real.0 <= real_indicator(s, x)
} by {
    if s.contains(x) {
        real_indicator_one_of_contains(s, x)
        zero_is_smaller_than_one[Real]
        lt_imp_lte[Real](Real.0, Real.1)
        Real.0 <= Real.1
        Real.0 <= real_indicator(s, x)
    }
    if not s.contains(x) {
        real_indicator_zero_of_not_contains(s, x)
        Real.0 <= Real.0
        Real.0 <= real_indicator(s, x)
    }
    s.contains(x) or not s.contains(x)
}

/// True if `sets` assigns a Lebesgue measurable set to every index in `values`.
define is_simple_function(values: FiniteSet[Real], sets: Real -> Set[Real]) -> Bool {
    forall(a: Real) {
        values.contains(a) implies lebesgue_measurable(sets(a))
    }
}

/// The contribution of the index `a` to the value of a simple function at `x`.
define value_term(sets: Real -> Set[Real], coeffs: Real -> Real, a: Real, x: Real) -> Real {
    coeffs(a) * real_indicator(sets(a), x)
}

/// The value of the simple function with indices `values`, sets `sets` and
/// coefficients `coeffs` at the point `x`.
define simple_function_value(values: FiniteSet[Real], sets: Real -> Set[Real], coeffs: Real -> Real, x: Real) -> Real {
    finite_set_sum(values, function(a: Real) { value_term(sets, coeffs, a, x) })
}

/// The contribution of the index `a` to the integral of a simple function.
define integral_term(sets: Real -> Set[Real], coeffs: Real -> Real, a: Real) -> Real {
    coeffs(a) * lebesgue_outer_measure(sets(a))
}

/// The Lebesgue integral of the simple function with indices `values`, sets
/// `sets` and coefficients `coeffs`: the sum over the indices of
/// `coeffs(a) * lambda(sets(a))`.
define simple_integral(values: FiniteSet[Real], sets: Real -> Set[Real], coeffs: Real -> Real) -> Real {
    finite_set_sum(values, function(a: Real) { integral_term(sets, coeffs, a) })
}

/// The value term of a simple function, truncated to zero outside the index
/// set.  This is nonnegative everywhere when the coefficients are nonnegative
/// on the index set.
define value_term_on(values: FiniteSet[Real], sets: Real -> Set[Real], coeffs: Real -> Real, a: Real, x: Real) -> Real {
    if values.contains(a) { value_term(sets, coeffs, a, x) } else { Real.0 }
}

/// The integral term of a simple function, truncated to zero outside the index
/// set.
define integral_term_on(values: FiniteSet[Real], sets: Real -> Set[Real], coeffs: Real -> Real, a: Real) -> Real {
    if values.contains(a) { integral_term(sets, coeffs, a) } else { Real.0 }
}

/// The value set of the indicator simple function: the singleton {one}.
let indicator_values: FiniteSet[Real] = fs_from_list(List.singleton[Real](Real.1))

/// The coefficient function of the indicator simple function: one at the
/// index one and zero elsewhere.
define indicator_coeffs(a: Real) -> Real {
    if a = Real.1 { Real.1 } else { Real.0 }
}

/// The level sets of the indicator simple function: the set `s` at the index
/// one and the empty set elsewhere.
define indicator_sets(s: Set[Real], a: Real) -> Set[Real] {
    if a = Real.1 { s } else { Set[Real].empty_set }
}

/// The index one belongs to the value set of the indicator simple function.
theorem indicator_values_contains_one {
    indicator_values.contains(Real.1)
} by {
    indicator_values = fs_from_list(List.singleton[Real](Real.1))
    fs_from_list(List.singleton[Real](Real.1)).underlying_set = list_set(List.singleton[Real](Real.1))
    list_set_contains_eq(List.singleton[Real](Real.1), Real.1)
    List.singleton[Real](Real.1).contains(Real.1)
    indicator_values.contains(Real.1)
}

/// An index of the indicator simple function is the value one.
theorem indicator_values_contains_imp_eq_one(a: Real) {
    indicator_values.contains(a) implies a = Real.1
} by {
    if indicator_values.contains(a) {
        indicator_values = fs_from_list(List.singleton[Real](Real.1))
        fs_from_list(List.singleton[Real](Real.1)).underlying_set = list_set(List.singleton[Real](Real.1))
        list_set_contains_eq(List.singleton[Real](Real.1), a)
        List.singleton[Real](Real.1).contains(a)
        singleton_contains_imp_eq[Real](Real.1, a)
        a = Real.1
    }
}

/// The indicator simple function of a Lebesgue measurable set is a simple
/// function.
theorem indicator_is_simple_function(s: Set[Real]) {
    lebesgue_measurable(s) implies is_simple_function(indicator_values, indicator_sets(s))
} by {
    if lebesgue_measurable(s) {
        forall(a: Real) {
            if indicator_values.contains(a) {
                indicator_values_contains_imp_eq_one(a)
                a = Real.1
                indicator_sets(s, a) = s
                lebesgue_measurable(s)
                lebesgue_measurable(indicator_sets(s, a))
            }
            indicator_values.contains(a) implies lebesgue_measurable(indicator_sets(s, a))
        }
        is_simple_function(indicator_values, indicator_sets(s))
    }
}

/// The value of the indicator simple function of `s` at `x` is the real-valued
/// indicator of `s` at `x`.
theorem simple_function_value_indicator(s: Set[Real], x: Real) {
    simple_function_value(indicator_values, indicator_sets(s), indicator_coeffs, x) = real_indicator(s, x)
} by {
    simple_function_value(indicator_values, indicator_sets(s), indicator_coeffs, x) =
        finite_set_sum(indicator_values, function(a: Real) { value_term(indicator_sets(s), indicator_coeffs, a, x) })
    indicator_values = fs_from_list(List.singleton[Real](Real.1))
    singleton_unique[Real](Real.1)
    List.singleton[Real](Real.1).is_unique
    finite_set_sum_eq_list_sum[Real, Real](List.singleton[Real](Real.1),
        function(a: Real) { value_term(indicator_sets(s), indicator_coeffs, a, x) })
    finite_set_sum(fs_from_list(List.singleton[Real](Real.1)),
        function(a: Real) { value_term(indicator_sets(s), indicator_coeffs, a, x) }) =
        sum[Real](map(List.singleton[Real](Real.1),
            function(a: Real) { value_term(indicator_sets(s), indicator_coeffs, a, x) }))
    finite_set_sum(indicator_values,
        function(a: Real) { value_term(indicator_sets(s), indicator_coeffs, a, x) }) =
        sum[Real](map(List.singleton[Real](Real.1),
            function(a: Real) { value_term(indicator_sets(s), indicator_coeffs, a, x) }))
    map_singleton[Real, Real](function(a: Real) { value_term(indicator_sets(s), indicator_coeffs, a, x) }, Real.1)
    map(List.singleton[Real](Real.1),
        function(a: Real) { value_term(indicator_sets(s), indicator_coeffs, a, x) }) =
        List.singleton[Real](value_term(indicator_sets(s), indicator_coeffs, Real.1, x))
    List.singleton[Real](value_term(indicator_sets(s), indicator_coeffs, Real.1, x)) =
        List.cons[Real](value_term(indicator_sets(s), indicator_coeffs, Real.1, x), List.nil[Real])
    sum[Real](List.cons[Real](value_term(indicator_sets(s), indicator_coeffs, Real.1, x), List.nil[Real])) =
        value_term(indicator_sets(s), indicator_coeffs, Real.1, x) + sum[Real](List.nil[Real])
    sum[Real](List.nil[Real]) = Real.0
    value_term(indicator_sets(s), indicator_coeffs, Real.1, x) + sum[Real](List.nil[Real]) =
        value_term(indicator_sets(s), indicator_coeffs, Real.1, x)
    sum[Real](map(List.singleton[Real](Real.1),
        function(a: Real) { value_term(indicator_sets(s), indicator_coeffs, a, x) })) =
        value_term(indicator_sets(s), indicator_coeffs, Real.1, x)
    value_term(indicator_sets(s), indicator_coeffs, Real.1, x) =
        indicator_coeffs(Real.1) * real_indicator(indicator_sets(s, Real.1), x)
    indicator_sets(s, Real.1) = s
    indicator_coeffs(Real.1) = Real.1
    value_term(indicator_sets(s), indicator_coeffs, Real.1, x) = Real.1 * real_indicator(s, x)
    mul_one_left(real_indicator(s, x))
    Real.1 * real_indicator(s, x) = real_indicator(s, x)
    value_term(indicator_sets(s), indicator_coeffs, Real.1, x) = real_indicator(s, x)
    simple_function_value(indicator_values, indicator_sets(s), indicator_coeffs, x) = real_indicator(s, x)
}

/// The integral of the indicator simple function of a set is its Lebesgue
/// outer measure.
theorem simple_integral_indicator(s: Set[Real]) {
    simple_integral(indicator_values, indicator_sets(s), indicator_coeffs) = lebesgue_outer_measure(s)
} by {
    simple_integral(indicator_values, indicator_sets(s), indicator_coeffs) =
        finite_set_sum(indicator_values, function(a: Real) { integral_term(indicator_sets(s), indicator_coeffs, a) })
    indicator_values = fs_from_list(List.singleton[Real](Real.1))
    singleton_unique[Real](Real.1)
    List.singleton[Real](Real.1).is_unique
    finite_set_sum_eq_list_sum[Real, Real](List.singleton[Real](Real.1),
        function(a: Real) { integral_term(indicator_sets(s), indicator_coeffs, a) })
    finite_set_sum(fs_from_list(List.singleton[Real](Real.1)),
        function(a: Real) { integral_term(indicator_sets(s), indicator_coeffs, a) }) =
        sum[Real](map(List.singleton[Real](Real.1),
            function(a: Real) { integral_term(indicator_sets(s), indicator_coeffs, a) }))
    finite_set_sum(indicator_values,
        function(a: Real) { integral_term(indicator_sets(s), indicator_coeffs, a) }) =
        sum[Real](map(List.singleton[Real](Real.1),
            function(a: Real) { integral_term(indicator_sets(s), indicator_coeffs, a) }))
    map_singleton[Real, Real](function(a: Real) { integral_term(indicator_sets(s), indicator_coeffs, a) }, Real.1)
    map(List.singleton[Real](Real.1),
        function(a: Real) { integral_term(indicator_sets(s), indicator_coeffs, a) }) =
        List.singleton[Real](integral_term(indicator_sets(s), indicator_coeffs, Real.1))
    sum[Real](map(List.singleton[Real](Real.1),
        function(a: Real) { integral_term(indicator_sets(s), indicator_coeffs, a) })) =
        sum[Real](List.singleton[Real](integral_term(indicator_sets(s), indicator_coeffs, Real.1)))
    List.singleton[Real](integral_term(indicator_sets(s), indicator_coeffs, Real.1)) =
        List.cons[Real](integral_term(indicator_sets(s), indicator_coeffs, Real.1), List.nil[Real])
    sum[Real](List.cons[Real](integral_term(indicator_sets(s), indicator_coeffs, Real.1), List.nil[Real])) =
        integral_term(indicator_sets(s), indicator_coeffs, Real.1) + sum[Real](List.nil[Real])
    sum[Real](List.nil[Real]) = Real.0
    integral_term(indicator_sets(s), indicator_coeffs, Real.1) + sum[Real](List.nil[Real]) =
        integral_term(indicator_sets(s), indicator_coeffs, Real.1)
    sum[Real](List.singleton[Real](integral_term(indicator_sets(s), indicator_coeffs, Real.1))) =
        integral_term(indicator_sets(s), indicator_coeffs, Real.1)
    integral_term(indicator_sets(s), indicator_coeffs, Real.1) =
        indicator_coeffs(Real.1) * lebesgue_outer_measure(indicator_sets(s, Real.1))
    indicator_sets(s, Real.1) = s
    indicator_coeffs(Real.1) = Real.1
    integral_term(indicator_sets(s), indicator_coeffs, Real.1) = Real.1 * lebesgue_outer_measure(s)
    mul_one_left(lebesgue_outer_measure(s))
    Real.1 * lebesgue_outer_measure(s) = lebesgue_outer_measure(s)
    integral_term(indicator_sets(s), indicator_coeffs, Real.1) = lebesgue_outer_measure(s)
    simple_integral(indicator_values, indicator_sets(s), indicator_coeffs) = lebesgue_outer_measure(s)
}

/// The value of a simple function is additive in the coefficients.
theorem simple_function_value_add(values: FiniteSet[Real], sets: Real -> Set[Real], c1: Real -> Real, c2: Real -> Real, x: Real) {
    simple_function_value(values, sets, add_fn(c1, c2), x) =
        simple_function_value(values, sets, c1, x) + simple_function_value(values, sets, c2, x)
} by {
    let f = function(a: Real) { value_term(sets, c1, a, x) }
    let g = function(a: Real) { value_term(sets, c2, a, x) }
    let h = function(a: Real) { value_term(sets, add_fn(c1, c2), a, x) }
    forall(a: Real) {
        h(a) = value_term(sets, add_fn(c1, c2), a, x)
        h(a) = add_fn(c1, c2, a) * real_indicator(sets(a), x)
        add_fn(c1, c2, a) = c1(a) + c2(a)
        mul_distrib_left(c1(a), c2(a), real_indicator(sets(a), x))
        (c1(a) + c2(a)) * real_indicator(sets(a), x) =
            c1(a) * real_indicator(sets(a), x) + c2(a) * real_indicator(sets(a), x)
        h(a) = c1(a) * real_indicator(sets(a), x) + c2(a) * real_indicator(sets(a), x)
        f(a) = value_term(sets, c1, a, x)
        f(a) = c1(a) * real_indicator(sets(a), x)
        g(a) = value_term(sets, c2, a, x)
        g(a) = c2(a) * real_indicator(sets(a), x)
        add_fn(f, g, a) = f(a) + g(a)
        h(a) = add_fn(f, g, a)
    }
    function_extensionality[Real, Real](h, add_fn(f, g))
    h = add_fn(f, g)
    finite_set_sum_add[Real, Real](values, f, g)
    finite_set_sum(values, f) + finite_set_sum(values, g) = finite_set_sum(values, add_fn(f, g))
    simple_function_value(values, sets, c1, x) = finite_set_sum(values, f)
    simple_function_value(values, sets, c2, x) = finite_set_sum(values, g)
    simple_function_value(values, sets, add_fn(c1, c2), x) = finite_set_sum(values, h)
    finite_set_sum(values, add_fn(f, g)) = finite_set_sum(values, h)
    finite_set_sum(values, f) + finite_set_sum(values, g) = finite_set_sum(values, h)
    simple_function_value(values, sets, add_fn(c1, c2), x) =
        simple_function_value(values, sets, c1, x) + simple_function_value(values, sets, c2, x)
}

/// The integral of a simple function is additive in the coefficients.
theorem simple_integral_add(values: FiniteSet[Real], sets: Real -> Set[Real], c1: Real -> Real, c2: Real -> Real) {
    simple_integral(values, sets, add_fn(c1, c2)) =
        simple_integral(values, sets, c1) + simple_integral(values, sets, c2)
} by {
    let f = function(a: Real) { integral_term(sets, c1, a) }
    let g = function(a: Real) { integral_term(sets, c2, a) }
    let h = function(a: Real) { integral_term(sets, add_fn(c1, c2), a) }
    forall(a: Real) {
        h(a) = integral_term(sets, add_fn(c1, c2), a)
        h(a) = add_fn(c1, c2, a) * lebesgue_outer_measure(sets(a))
        add_fn(c1, c2, a) = c1(a) + c2(a)
        mul_distrib_left(c1(a), c2(a), lebesgue_outer_measure(sets(a)))
        (c1(a) + c2(a)) * lebesgue_outer_measure(sets(a)) =
            c1(a) * lebesgue_outer_measure(sets(a)) + c2(a) * lebesgue_outer_measure(sets(a))
        h(a) = c1(a) * lebesgue_outer_measure(sets(a)) + c2(a) * lebesgue_outer_measure(sets(a))
        f(a) = integral_term(sets, c1, a)
        f(a) = c1(a) * lebesgue_outer_measure(sets(a))
        g(a) = integral_term(sets, c2, a)
        g(a) = c2(a) * lebesgue_outer_measure(sets(a))
        add_fn(f, g, a) = f(a) + g(a)
        h(a) = add_fn(f, g, a)
    }
    function_extensionality[Real, Real](h, add_fn(f, g))
    h = add_fn(f, g)
    finite_set_sum_add[Real, Real](values, f, g)
    finite_set_sum(values, f) + finite_set_sum(values, g) = finite_set_sum(values, add_fn(f, g))
    simple_integral(values, sets, c1) = finite_set_sum(values, f)
    simple_integral(values, sets, c2) = finite_set_sum(values, g)
    simple_integral(values, sets, add_fn(c1, c2)) = finite_set_sum(values, h)
    finite_set_sum(values, add_fn(f, g)) = finite_set_sum(values, h)
    finite_set_sum(values, f) + finite_set_sum(values, g) = finite_set_sum(values, h)
    simple_integral(values, sets, add_fn(c1, c2)) =
        simple_integral(values, sets, c1) + simple_integral(values, sets, c2)
}

/// The integral of a simple function is homogeneous in the coefficients.
theorem simple_integral_scalar_mul(values: FiniteSet[Real], sets: Real -> Set[Real], c: Real, coeffs: Real -> Real) {
    simple_integral(values, sets, mul_fn(c, coeffs)) = c * simple_integral(values, sets, coeffs)
} by {
    let f = function(a: Real) { integral_term(sets, coeffs, a) }
    let h = function(a: Real) { integral_term(sets, mul_fn(c, coeffs), a) }
    forall(a: Real) {
        h(a) = integral_term(sets, mul_fn(c, coeffs), a)
        h(a) = mul_fn(c, coeffs, a) * lebesgue_outer_measure(sets(a))
        mul_fn(c, coeffs, a) = c * coeffs(a)
        mul_assoc(c, coeffs(a), lebesgue_outer_measure(sets(a)))
        c * (coeffs(a) * lebesgue_outer_measure(sets(a))) =
            (c * coeffs(a)) * lebesgue_outer_measure(sets(a))
        h(a) = c * (coeffs(a) * lebesgue_outer_measure(sets(a)))
        f(a) = integral_term(sets, coeffs, a)
        f(a) = coeffs(a) * lebesgue_outer_measure(sets(a))
        c * f(a) = c * (coeffs(a) * lebesgue_outer_measure(sets(a)))
        mul_fn(c, f, a) = c * f(a)
        h(a) = mul_fn(c, f, a)
    }
    function_extensionality[Real, Real](h, mul_fn(c, f))
    h = mul_fn(c, f)
    finite_set_sum_scalar_mul[Real, Real](c, values, f)
    c * finite_set_sum(values, f) = finite_set_sum(values, mul_fn(c, f))
    simple_integral(values, sets, coeffs) = finite_set_sum(values, f)
    simple_integral(values, sets, mul_fn(c, coeffs)) = finite_set_sum(values, h)
    finite_set_sum(values, mul_fn(c, f)) = finite_set_sum(values, h)
    c * simple_integral(values, sets, coeffs) = finite_set_sum(values, h)
    simple_integral(values, sets, mul_fn(c, coeffs)) = c * simple_integral(values, sets, coeffs)
}

/// The value of a simple function with nonnegative coefficients is nonnegative.
theorem simple_function_value_nonneg(values: FiniteSet[Real], sets: Real -> Set[Real], coeffs: Real -> Real, x: Real) {
    (forall(a: Real) { values.contains(a) implies Real.0 <= coeffs(a) })
        implies Real.0 <= simple_function_value(values, sets, coeffs, x)
} by {
    if forall(a: Real) { values.contains(a) implies Real.0 <= coeffs(a) } {
        forall(a: Real) {
            if values.contains(a) {
                Real.0 <= coeffs(a)
                coeffs(a) >= Real.0
                real_indicator_nonneg(sets(a), x)
                Real.0 <= real_indicator(sets(a), x)
                real_indicator(sets(a), x) >= Real.0
                mul_nonneg(coeffs(a), real_indicator(sets(a), x))
                coeffs(a) * real_indicator(sets(a), x) >= Real.0
                Real.0 <= coeffs(a) * real_indicator(sets(a), x)
                value_term(sets, coeffs, a, x) = coeffs(a) * real_indicator(sets(a), x)
                Real.0 <= value_term(sets, coeffs, a, x)
            }
        }
        forall(a: Real) {
            if values.contains(a) {
                value_term_on(values, sets, coeffs, a, x) = value_term(sets, coeffs, a, x)
                Real.0 <= value_term_on(values, sets, coeffs, a, x)
            }
            if not values.contains(a) {
                value_term_on(values, sets, coeffs, a, x) = Real.0
                Real.0 <= Real.0
                Real.0 <= value_term_on(values, sets, coeffs, a, x)
            }
            values.contains(a) or not values.contains(a)
            Real.0 <= value_term_on(values, sets, coeffs, a, x)
        }
        finite_set_sum_nonneg[Real](values, function(a: Real) { value_term_on(values, sets, coeffs, a, x) })
        Real.0 <= finite_set_sum(values, function(a: Real) { value_term_on(values, sets, coeffs, a, x) })
        forall(a: Real) {
            if values.contains(a) {
                value_term_on(values, sets, coeffs, a, x) = value_term(sets, coeffs, a, x)
            }
        }
        finite_set_sum_eq_of_pointwise_on_set[Real, Real](values,
            function(a: Real) { value_term_on(values, sets, coeffs, a, x) },
            function(a: Real) { value_term(sets, coeffs, a, x) })
        finite_set_sum(values, function(a: Real) { value_term_on(values, sets, coeffs, a, x) }) =
            finite_set_sum(values, function(a: Real) { value_term(sets, coeffs, a, x) })
        Real.0 <= finite_set_sum(values, function(a: Real) { value_term(sets, coeffs, a, x) })
        simple_function_value(values, sets, coeffs, x) =
            finite_set_sum(values, function(a: Real) { value_term(sets, coeffs, a, x) })
        Real.0 <= simple_function_value(values, sets, coeffs, x)
    }
}

/// The integral of a simple function with nonnegative coefficients is
/// nonnegative.
theorem simple_integral_nonneg(values: FiniteSet[Real], sets: Real -> Set[Real], coeffs: Real -> Real) {
    (forall(a: Real) { values.contains(a) implies Real.0 <= coeffs(a) })
        and (forall(a: Real) { values.contains(a) implies lebesgue_cost_has_infimum(sets(a)) })
        implies Real.0 <= simple_integral(values, sets, coeffs)
} by {
    if forall(a: Real) { values.contains(a) implies Real.0 <= coeffs(a) }
        and forall(a: Real) { values.contains(a) implies lebesgue_cost_has_infimum(sets(a)) } {
        forall(a: Real) {
            if values.contains(a) {
                Real.0 <= coeffs(a)
                coeffs(a) >= Real.0
                lebesgue_cost_has_infimum(sets(a))
                lebesgue_outer_measure_nonneg(sets(a))
                Real.0 <= lebesgue_outer_measure(sets(a))
                lebesgue_outer_measure(sets(a)) >= Real.0
                mul_nonneg(coeffs(a), lebesgue_outer_measure(sets(a)))
                coeffs(a) * lebesgue_outer_measure(sets(a)) >= Real.0
                Real.0 <= coeffs(a) * lebesgue_outer_measure(sets(a))
                integral_term(sets, coeffs, a) = coeffs(a) * lebesgue_outer_measure(sets(a))
                Real.0 <= integral_term(sets, coeffs, a)
            }
        }
        forall(a: Real) {
            if values.contains(a) {
                integral_term_on(values, sets, coeffs, a) = integral_term(sets, coeffs, a)
                Real.0 <= integral_term_on(values, sets, coeffs, a)
            }
            if not values.contains(a) {
                integral_term_on(values, sets, coeffs, a) = Real.0
                Real.0 <= Real.0
                Real.0 <= integral_term_on(values, sets, coeffs, a)
            }
            values.contains(a) or not values.contains(a)
            Real.0 <= integral_term_on(values, sets, coeffs, a)
        }
        finite_set_sum_nonneg[Real](values, function(a: Real) { integral_term_on(values, sets, coeffs, a) })
        Real.0 <= finite_set_sum(values, function(a: Real) { integral_term_on(values, sets, coeffs, a) })
        forall(a: Real) {
            if values.contains(a) {
                integral_term_on(values, sets, coeffs, a) = integral_term(sets, coeffs, a)
            }
        }
        finite_set_sum_eq_of_pointwise_on_set[Real, Real](values,
            function(a: Real) { integral_term_on(values, sets, coeffs, a) },
            function(a: Real) { integral_term(sets, coeffs, a) })
        finite_set_sum(values, function(a: Real) { integral_term_on(values, sets, coeffs, a) }) =
            finite_set_sum(values, function(a: Real) { integral_term(sets, coeffs, a) })
        Real.0 <= finite_set_sum(values, function(a: Real) { integral_term(sets, coeffs, a) })
        simple_integral(values, sets, coeffs) =
            finite_set_sum(values, function(a: Real) { integral_term(sets, coeffs, a) })
        Real.0 <= simple_integral(values, sets, coeffs)
    }
}

// TODO (full linearity): for simple functions with possibly different index
// sets, `f = sum_{a in V} c1(a) * 1_{A_a}` and `g = sum_{b in W} c2(b) * 1_{B_b}`,
// the pointwise sum `f + g` is again a simple function, represented over the
// index set `V x W` with sets `A_a.intersection(B_b)` and coefficients
// `c1(a) + c2(b)`, and the integral satisfies
//     simple_integral(VxW, refinement, coeffs_fg) =
//         simple_integral(V, A, c1) + simple_integral(W, B, c2)
// The coefficient additivity `simple_integral_add` above covers the case of a
// common index set.  The general statement additionally needs the finite
// additivity of the Lebesgue outer measure on disjoint measurable sets
// (λ(A ∪ B) = λ(A) + λ(B) for measurable A, B with A ∩ B = ∅), which is not
// yet available in lebesgue.ac (cf. the TODOs on countable subadditivity
// there); the indicator case ∫(1_A + 1_B) = λ(A) + λ(B) reduces to exactly
// that additivity once 1_A + 1_B is rewritten over the partition
// (A ∩ B, A \ B, B \ A).
