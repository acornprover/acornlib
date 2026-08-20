/// Public interface for analysis.

from algebra.add_comm_monoid_rearrange import add_swap_inner
from algebra.add_ordered_group import add_le_add, add_le_iff_le_sub_right,
    add_lt_add_left, add_lt_add_right, minus_flips_inequality
from algebra.field.field import mul_not_zero
from algebra.module.normed_add_comm_group import NormedAddCommGroup
from algebra.module.normed_field import NormedField
from data.basic.functions import compose, compose_assoc, compose_injective_fn,
    function_extensionality, identity_fn, is_bijection_fn, is_injective_fn,
    is_surjective_fn
from data.basic.logic import by_contradiction, exists_intro, not_forall_imp_exists_not,
    not_implies
from data.basic.set import Set, compl_contains_eq, compl_of_compl_is_self,
    difference_contains_eq, empty_set_compl_is_universal, empty_set_contains_eq,
    empty_set_is_always_subset, intersection_assoc, intersection_comm,
    intersection_contains_eq, intersection_contains_intro, intersection_contains_left,
    intersection_with_empty_is_empty, intersection_with_universal_is_self,
    is_saturated_under, preimage_contains, set_eq_of_contains_at_eq, set_ext, set_image,
    set_image_contains_witness, set_is_disjoint_compl, set_preimage, set_preimage_compl,
    set_preimage_compose, set_preimage_contains_eq, set_preimage_intersection,
    sets_subset_contain_union, sets_subset_intersection, sets_subset_union,
    singleton_contains_eq, subset_antisymm, subset_contains, subset_contains_eq,
    subset_refl, subset_trans, union_comm, union_compl_is_universal, union_contains_eq,
    universal_set_compl_is_empty, universal_set_contains_eq
from data.basic.set_lattice import set_inf, set_subset_inf_of_subset_left_right
from int import Int, abs, abs_zero_imp_zero, add_sub_lt_right_of_lt, lte_add_left,
    lte_from_nat, pos_is_not_neg, sub_nonnegative_of_lte, zero_lt_pos
from list import List
from nat import Nat, add_comm, add_one_right, add_sub, add_suc_right, from_nat,
    from_nat_add, from_nat_mul, from_nat_one, from_nat_zero, lt_add_suc, lt_and_lte,
    lt_imp_lte_suc, lt_mul_both, lt_suc, lte_and_lt, lte_ref, lte_trans, sub_pos
from order import closed_interval, is_monotone, lt_iff_lte_and_ne, lt_imp_lte, lt_imp_ne,
    lt_lte_trans, lt_min_iff, lt_not_ref, lt_of_lt_of_lte, lt_or_lte, lt_trans,
    lte_antisymm, lte_lt_trans, lte_min_of_bounds, min_lte_left, min_lte_right,
    monotone_apply, not_lt_self, not_lte_imp_gt, open_interval
from order_set import closed_interval_set, closed_interval_set_contains_eq,
    closed_interval_set_le_upper, closed_interval_set_lower_le, open_interval_set,
    open_interval_set_contains_eq
from ordered_field import inverse_of_positive_is_positive,
    inverse_on_positive_flips_inequality, mul_lt_mul_of_pos_right
from pair import Pair, pair_ext, pair_new_first, pair_new_second
from rat import Rat, from_int_num, lt_some_nat, lte_from_int, mul_int_eq_int_mul
from real import Real, abs_gte_zero, add_assoc, add_from_int, add_neg_eq_zero,
    add_rat_eps_between, add_seq, add_zero_left, add_zero_right, bounds_imp_close,
    cauchy_imp_converges_to_limit, cauchy_imp_exists_limit, close_imp_bounds,
    completeness, const_converges, const_limit, continuous, continuous_at,
    continuous_condition, convergent_converges_to_limit, converges, converges_to,
    converges_to_imp_converges, converges_to_unique, div_cancel_common, div_le_of_mul_le,
    div_lt_mul_pos, eps_lt_half, eventual_ub, exists_small_mul, floor,
    from_nat_is_from_rat, from_rat_maintains_lt, from_rat_maintains_lte, from_rat_pos,
    gt_imp_not_lte, gte_imp_not_lt, has_upper_bound,
    increasing_convergent_bounded_by_limit, intermediate_value_closed_interval,
    is_cauchy_seq, is_increasing, is_lower_bound, is_nonempty, is_set_supremum,
    is_set_upper_bound, is_upper_bound, limit, limit_add_seq, lt_add_converse,
    lt_add_one, lt_add_right, lt_imp_minus_pos, lt_some_int_cancel, lte_abs,
    lte_add_right, lte_both_ways_imp_eq, lte_mul_nonneg_left, lte_mul_nonneg_right,
    lte_or_gte, lte_self, monotone_convergence_principle, mul_abs, mul_div_cancel,
    mul_from_rat, mul_inverse, mul_le_mul_pos_left, mul_one_left, mul_one_over,
    mul_sub_distrib_left, neg_distrib, neg_lt_zero, neg_neg, neg_pos_is_neg, neg_zero,
    not_lt_imp_gte, only_abs_zero_eq_zero, rat_upper, seq_lte, seq_lte_preserves_limit,
    sub_cancels, supremum_subset_le, tail_bound, tail_bound_implies_is_close,
    triangle_ineq, ub_imp_limit_lte
from sum_type import Sum

numerals Real
typeclass M: MetricSpace {
    /// Every metric space must have a distance function
    distance: (M, M) -> Real

    /// Rule: the distance from a point to itself is zero
    self_distance_is_zero(x: M) {
        x.distance(x) = 0
    }

    /// Rule: distance zero means the points are equal
    dist_zero_imp_eq(x: M, y: M) {
        x.distance(y) = 0 implies x = y
    }

    /// Rule: distance must be symmetric
    symmetric(x: M, y: M) {
        x.distance(y) = y.distance(x)
    }

    /// Rule: distance must satisfy the triangle inequality
    triangle(x: M, y: M, z: M) {
        x.distance(z) <= x.distance(y) + y.distance(z)
    }
}

define cauchy_metric_bound[M: MetricSpace](q: Nat -> M, n: Nat, eps: Real) -> Bool {
    forall(i: Nat, j: Nat) {
        n <= i and n <= j implies q(i).distance(q(j)) < eps
    }
}

define is_cauchy_metric[M: MetricSpace](q: Nat -> M) -> Bool {
    forall(eps: Real) {
        eps.is_positive implies exists(n: Nat) {
            cauchy_metric_bound(q, n, eps)
        }
    }
}

define metric_tail_bound[M: MetricSpace](q: Nat -> M, a: M, n: Nat, eps: Real) -> Bool {
    forall(i: Nat) {
        n <= i implies q(i).distance(a) < eps
    }
}

define tendsto_metric[M: MetricSpace](q: Nat -> M, a: M) -> Bool {
    forall(eps: Real) {
        eps.is_positive implies exists(n: Nat) {
            metric_tail_bound(q, a, n, eps)
        }
    }
}

typeclass M: CompleteMetricSpace extends MetricSpace {
    /// Rule: every Cauchy sequence has a limit.
    cauchy_metric_imp_tendsto(q: Nat -> M) {
        is_cauchy_metric(q) implies exists(a: M) {
            tendsto_metric(q, a)
        }
    }
}

define is_local_equiv_data[A, B](source: Set[A], target: Set[B],
    to_fun: A -> B, inv_fun: B -> A) -> Bool {
    forall(x: A) {
        source.contains(x) implies target.contains(to_fun(x))
    } and forall(y: B) {
        target.contains(y) implies source.contains(inv_fun(y))
    } and forall(x: A) {
        source.contains(x) implies inv_fun(to_fun(x)) = x
    } and forall(y: B) {
        target.contains(y) implies to_fun(inv_fun(y)) = y
    }
}

structure LocalEquiv[A, B] {
    /// The set on which the forward map is defined.
    source: Set[A]

    /// The set on which the inverse map is defined.
    target: Set[B]

    /// The forward partial map.
    to_fun: A -> B

    /// The inverse partial map.
    inv_fun: B -> A
} constraint {
    is_local_equiv_data(source, target, to_fun, inv_fun)
}

define big_union_contains[T](c: Set[T] -> Bool, x: T) -> Bool {
    exists(s: Set[T]) {
        c(s) and s.contains(x)
    }
}

define big_union[T](c: Set[T] -> Bool) -> Set[T] {
    Set[T].new(big_union_contains(c))
}

typeclass T: TopologicalSpace {
    /// True if the given subset is open in the topology.
    is_open: Set[T] -> Bool

    /// Rule: the empty set is open.
    open_empty {
        T.is_open(Set[T].empty_set)
    }

    /// Rule: the whole space is open.
    open_universal {
        T.is_open(Set[T].universal_set)
    }

    /// Rule: the intersection of two open sets is open.
    open_inter(s: Set[T], t: Set[T]) {
        T.is_open(s) and T.is_open(t) implies T.is_open(s.intersection(t))
    }

    /// Rule: the union of any family of open sets is open.
    open_big_union(c: Set[T] -> Bool) {
        (forall(s: Set[T]) { c(s) implies T.is_open(s) })
            implies T.is_open(big_union(c))
    }
}

numerals Real
define in_open_ball[M: MetricSpace](x: M, r: Real, y: M) -> Bool {
    y.distance(x) < r
}

define open_ball[M: MetricSpace](x: M, r: Real) -> Set[M] {
    Set[M].new(in_open_ball(x, r))
}

define is_open_metric[M: MetricSpace](s: Set[M]) -> Bool {
    forall(x: M) {
        s.contains(x) implies exists(r: Real) {
            r.is_positive and open_ball(x, r).subset(s)
        }
    }
}

typeclass M: MetricTopologicalSpace extends MetricSpace, TopologicalSpace {
    /// Compatibility axiom: topological openness agrees with metric openness.
    topological_open_eq_metric_open(s: Set[M]) {
        M.is_open(s) = is_open_metric(s)
    }
}

define real_distance(x: Real, y: Real) -> Real {
    (x - y).abs
}

define real_is_interior_point(s: Set[Real], x: Real) -> Bool {
    s.contains(x) and exists(eps: Real) {
        eps.is_positive and forall(y: Real) {
            y.is_close(x, eps) implies s.contains(y)
        }
    }
}

define real_is_open(s: Set[Real]) -> Bool {
    forall(x: Real) {
        s.contains(x) implies real_is_interior_point(s, x)
    }
}

define real_norm(x: Real) -> Real {
    x.abs
}

instance Real: MetricSpace {
    let distance: (Real, Real) -> Real = real_distance
}

instance Real: CompleteMetricSpace

instance Real: TopologicalSpace {
    let is_open: Set[Real] -> Bool = real_is_open
}

instance Real: MetricTopologicalSpace

instance Real: NormedAddCommGroup {
    let norm: Real -> Real = real_norm
}

instance Real: NormedField


theorem add_one_minus(x: Real) {
    x + (Real.1 - x) = Real.1
}

define is_bounded_above(a: Nat -> Real) -> Bool {
    exists(b: Real) {
        is_upper_bound(a, b)
    }
}

theorem add_seq_bounded_above(a: Nat -> Real, b: Nat -> Real) {
    is_bounded_above(a) and is_bounded_above(b) implies is_bounded_above(add_seq(a, b))
}

define is_bounded_below(a: Nat -> Real) -> Bool {
    exists(b: Real) {
        is_lower_bound(a, b)
    }
}

theorem add_seq_bounded_below(a: Nat -> Real, b: Nat -> Real) {
    is_bounded_below(a) and is_bounded_below(b) implies is_bounded_below(add_seq(a, b))
}

define basis_covers[T](b: Set[T] -> Bool) -> Bool {
    forall(x: T) {
        exists(u: Set[T]) {
            b(u) and u.contains(x)
        }
    }
}

define basis_filters[T](b: Set[T] -> Bool) -> Bool {
    forall(u: Set[T], v: Set[T], x: T) {
        b(u) and b(v) and u.contains(x) and v.contains(x) implies exists(w: Set[T]) {
            b(w) and w.contains(x) and w.subset(u.intersection(v))
        }
    }
}

define generated_open[T](b: Set[T] -> Bool, s: Set[T]) -> Bool {
    forall(x: T) {
        s.contains(x) implies exists(u: Set[T]) {
            b(u) and u.contains(x) and u.subset(s)
        }
    }
}

theorem basis_member_is_open[T](b: Set[T] -> Bool, u: Set[T]) {
    b(u) implies generated_open(b, u)
}

define box_contains[X, Y](u: Set[X], v: Set[Y], p: Pair[X, Y]) -> Bool {
    u.contains(p.first) and v.contains(p.second)
}

define box[X, Y](u: Set[X], v: Set[Y]) -> Set[Pair[X, Y]] {
    Set[Pair[X, Y]].new(box_contains(u, v))
}

theorem box_contains_eq[X, Y](u: Set[X], v: Set[Y], p: Pair[X, Y]) {
    box(u, v).contains(p) = (u.contains(p.first) and v.contains(p.second))
}

define is_box[X: TopologicalSpace, Y: TopologicalSpace](b: Set[Pair[X, Y]]) -> Bool {
    exists(u: Set[X], v: Set[Y]) {
        X.is_open(u) and Y.is_open(v) and b = box(u, v)
    }
}

define is_open_in_product[X: TopologicalSpace, Y: TopologicalSpace](
    s: Set[Pair[X, Y]]) -> Bool {
    generated_open(is_box[X, Y], s)
}

theorem box_is_open_in_product[X: TopologicalSpace, Y: TopologicalSpace](
    u: Set[X], v: Set[Y]) {
    X.is_open(u) and Y.is_open(v) implies is_open_in_product[X, Y](box(u, v))
}

numerals Real
theorem cauchy_metric_bound_at[M: MetricSpace](q: Nat -> M, n: Nat, eps: Real, i: Nat, j: Nat) {
    cauchy_metric_bound(q, n, eps) and n <= i and n <= j implies q(i).distance(q(j)) < eps
}

theorem cauchy_metric_bound_from_distances[M: MetricSpace](q: Nat -> M, n: Nat, eps: Real) {
    (forall(i: Nat, j: Nat) {
        n <= i and n <= j implies q(i).distance(q(j)) < eps
    }) implies cauchy_metric_bound(q, n, eps)
}

numerals Nat
define frac(x: Real) -> Real {
    x - Real.from_int(floor(x))
}

define circular_distance(x: Real) -> Real {
    frac(x).min(Real.1 - frac(x))
}

theorem cd_bound(x: Real, b: Real) {
    Real.0 <= x and x < Real.1 and b <= x and b <= Real.1 - x
    implies b <= circular_distance(x)
}

theorem cd_bound_frac(k: Nat, n: Nat) {
    Nat.1 <= k and k < n implies
        Real.1 / from_nat[Real](n) <= circular_distance(from_nat[Real](k) / from_nat[Real](n))
}

theorem circular_distance_of_unit_interval(x: Real) {
    Real.0 <= x and x < Real.1 implies circular_distance(x) = x.min(Real.1 - x)
}

numerals Real
define in_closed_ball[M: MetricSpace](x: M, r: Real, y: M) -> Bool {
    y.distance(x) <= r
}

define closed_ball[M: MetricSpace](x: M, r: Real) -> Set[M] {
    Set[M].new(in_closed_ball(x, r))
}

define is_closed[T: TopologicalSpace](s: Set[T]) -> Bool {
    T.is_open(s.c)
}

define closure_contains[T: TopologicalSpace](s: Set[T], x: T) -> Bool {
    forall(c: Set[T]) {
        is_closed(c) and s.subset(c) implies c.contains(x)
    }
}

define closure[T: TopologicalSpace](s: Set[T]) -> Set[T] {
    Set[T].new(closure_contains(s))
}

define is_continuous[X: TopologicalSpace, Y: TopologicalSpace](f: X -> Y) -> Bool {
    forall(v: Set[Y]) {
        Y.is_open(v) implies X.is_open(set_preimage(f, v))
    }
}

define continuous_on_closed(f: Real -> Real, lower: Real, upper: Real) -> Bool {
    forall(x: Real) {
        closed_interval_set(lower, upper).contains(x) implies continuous_at(f, x)
    }
}

theorem continuous_open_preimage[X: TopologicalSpace, Y: TopologicalSpace](
    f: X -> Y, v: Set[Y]) {
    is_continuous(f) and Y.is_open(v) implies X.is_open(set_preimage(f, v))
}

numerals Real
theorem distance_non_negative[M: MetricSpace](x: M, y: M) {
    not x.distance(y).is_negative
}

theorem distance_self[M: MetricSpace](x: M) {
    x.distance(x) = 0
}

theorem div_le_div_pos(a: Real, b: Real, c: Real) {
    a <= b and Real.0 < c implies a / c <= b / c
}

define is_open_cover[T: TopologicalSpace](c: Set[T] -> Bool, s: Set[T]) -> Bool {
    (forall(u: Set[T]) { c(u) implies T.is_open(u) }) and s.subset(big_union(c))
}

define is_subfamily[X](c: Set[X] -> Bool, items: List[Set[X]]) -> Bool {
    match items {
        List.nil {
            true
        }
        List.cons(head, tail) {
            c(head) and is_subfamily(c, tail)
        }
    }
}

define list_union_of_sets[X](items: List[Set[X]]) -> Set[X] {
    match items {
        List.nil {
            Set[X].empty_set
        }
        List.cons(head, tail) {
            head.union(list_union_of_sets(tail))
        }
    }
}

define is_compact[T: TopologicalSpace](s: Set[T]) -> Bool {
    forall(c: Set[T] -> Bool) {
        is_open_cover[T](c, s) implies exists(items: List[Set[T]]) {
            is_subfamily[T](c, items) and s.subset(list_union_of_sets(items))
        }
    }
}

define finitely_coverable(c: Set[Real] -> Bool, a: Real, x: Real) -> Bool {
    exists(items: List[Set[Real]]) {
        is_subfamily[Real](c, items) and closed_interval_set(a, x).subset(list_union_of_sets[Real](items))
    }
}

numerals Int
theorem floor_is_greatest(x: Real, n: Int) {
    Real.from_int(n) <= x implies n <= floor(x)
}

numerals Nat
theorem floor_zero_of_unit_interval(x: Real) {
    Real.0 <= x and x < Real.1 implies floor(x) = Int.0
}

theorem frac_of_unit_interval(x: Real) {
    Real.0 <= x and x < Real.1 implies frac(x) = x
}

theorem from_nat_frac_lt_one(k: Nat, n: Nat) {
    k < n implies from_nat[Real](k) / from_nat[Real](n) < Real.1
}

theorem from_nat_real_lt(m: Nat, n: Nat) {
    m < n implies from_nat[Real](m) < from_nat[Real](n)
}

numerals Real
theorem from_nat_real_lte(m: Nat, n: Nat) {
    m <= n implies from_nat[Real](m) <= from_nat[Real](n)
}

theorem from_nat_real_nonnegative(n: Nat) {
    Real.0 <= from_nat[Real](n)
}

numerals Nat
theorem from_nat_real_positive(n: Nat) {
    Nat.0 < n implies Real.0 < from_nat[Real](n)
}

theorem generated_open_big_union[T](b: Set[T] -> Bool, c: Set[T] -> Bool) {
    (forall(s: Set[T]) { c(s) implies generated_open(b, s) })
        implies generated_open(b, big_union(c))
}

theorem generated_open_imp_is_open_in_product[X: TopologicalSpace, Y: TopologicalSpace](
    s: Set[Pair[X, Y]]) {
    generated_open(is_box[X, Y], s) implies is_open_in_product[X, Y](s)
}

define is_topological_basis[T](b: Set[T] -> Bool) -> Bool {
    basis_covers(b) and basis_filters(b)
}

theorem generated_open_inter[T](b: Set[T] -> Bool, s: Set[T], t: Set[T]) {
    is_topological_basis(b) and generated_open(b, s) and generated_open(b, t)
        implies generated_open(b, s.intersection(t))
}

theorem generated_open_witness[T](b: Set[T] -> Bool, s: Set[T], x: T) {
    generated_open(b, s) and s.contains(x) implies exists(u: Set[T]) {
        b(u) and u.contains(x) and u.subset(s)
    }
}

theorem heine_borel_closed_interval_is_compact(a: Real, b: Real) {
    is_compact[Real](closed_interval_set(a, b))
}

numerals Int
theorem int_lt_imp_add_one_lte(a: Int, b: Int) {
    a < b implies a + Int.1 <= b
}

define interior_contains[T: TopologicalSpace](s: Set[T], x: T) -> Bool {
    exists(u: Set[T]) {
        T.is_open(u) and u.subset(s) and u.contains(x)
    }
}

define interior[T: TopologicalSpace](s: Set[T]) -> Set[T] {
    Set[T].new(interior_contains(s))
}

define open_subsets_family[T: TopologicalSpace](s: Set[T], u: Set[T]) -> Bool {
    T.is_open(u) and u.subset(s)
}

define is_neighborhood[T: TopologicalSpace](n: Set[T], x: T) -> Bool {
    exists(u: Set[T]) {
        T.is_open(u) and u.contains(x) and u.subset(n)
    }
}

numerals Real
define is_bounded[M: MetricSpace](s: Set[M]) -> Bool {
    exists(x: M, r: Real) {
        s.subset(closed_ball(x, r))
    }
}

theorem is_bounded_above_intro(a: Nat -> Real, b: Real) {
    is_upper_bound(a, b) implies is_bounded_above(a)
}

theorem is_bounded_below_intro(a: Nat -> Real, b: Real) {
    is_lower_bound(a, b) implies is_bounded_below(a)
}

theorem is_box_is_topological_basis[X: TopologicalSpace, Y: TopologicalSpace] {
    is_topological_basis(is_box[X, Y])
}

numerals Real
define is_fixed_point[T](f: T -> T, x: T) -> Bool {
    f(x) = x
}

numerals Real
define is_isometry[M: MetricSpace, N: MetricSpace](f: M -> N) -> Bool {
    forall(x: M, y: M) {
        f(x).distance(f(y)) = x.distance(y)
    }
}

define lipschitz_bound[M: MetricSpace, N: MetricSpace](k: Real, f: M -> N) -> Bool {
    forall(x: M, y: M) {
        f(x).distance(f(y)) <= k * x.distance(y)
    }
}

define is_lipschitz_with[M: MetricSpace, N: MetricSpace](k: Real, f: M -> N) -> Bool {
    not k.is_negative and lipschitz_bound(k, f)
}

define is_lipschitz[M: MetricSpace, N: MetricSpace](f: M -> N) -> Bool {
    exists(k: Real) {
        is_lipschitz_with(k, f)
    }
}

theorem is_open_in_product_imp_generated_open[X: TopologicalSpace, Y: TopologicalSpace](
    s: Set[Pair[X, Y]]) {
    is_open_in_product[X, Y](s) implies generated_open(is_box[X, Y], s)
}

define is_unbounded(f: Nat -> Nat) -> Bool {
    forall(bound: Nat) {
        exists(n: Nat) {
            bound < f(n)
        }
    }
}

numerals Real
theorem isometry_is_injective_fn[M: MetricSpace, N: MetricSpace](f: M -> N) {
    is_isometry(f) implies is_injective_fn(f)
}

theorem isometry_is_lipschitz[M: MetricSpace, N: MetricSpace](f: M -> N) {
    is_isometry(f) implies is_lipschitz(f)
}

theorem isometry_is_lipschitz_with_one[M: MetricSpace, N: MetricSpace](f: M -> N) {
    is_isometry(f) implies is_lipschitz_with(1, f)
}

theorem isometry_preserves_cauchy_metric[M: MetricSpace, N: MetricSpace](
    f: M -> N, q: Nat -> M
) {
    is_isometry(f) and is_cauchy_metric(q) implies is_cauchy_metric(compose(f, q))
}

theorem isometry_preserves_tendsto_metric[M: MetricSpace, N: MetricSpace](
    f: M -> N, q: Nat -> M, a: M
) {
    is_isometry(f) and tendsto_metric(q, a) implies tendsto_metric(compose(f, q), f(a))
}

define iterate[T](f: T -> T, n: Nat, x: T) -> T {
    match n {
        Nat.zero {
            x
        }
        Nat.suc(pred) {
            f(iterate(f, pred, x))
        }
    }
}

theorem iterate_one[T](f: T -> T, x: T) {
    iterate(f, Nat.1, x) = f(x)
}

theorem iterate_suc[T](f: T -> T, n: Nat, x: T) {
    iterate(f, n.suc, x) = f(iterate(f, n, x))
}

theorem iterate_suc_inner[T](f: T -> T, n: Nat, x: T) {
    iterate(f, n.suc, x) = iterate(f, n, f(x))
}

theorem iterate_zero[T](f: T -> T, x: T) {
    iterate(f, Nat.0, x) = x
}

define tail_value(a: Nat -> Real, n: Nat, x: Real) -> Bool {
    exists(k: Nat) {
        n <= k and a(k) = x
    }
}

define tail_set(a: Nat -> Real, n: Nat) -> Set[Real] {
    Set[Real].new(tail_value(a, n))
}

let tail_sup(a: Nat -> Real, n: Nat) -> result: Real satisfy {
    is_bounded_above(a) implies is_set_supremum(tail_set(a, n), result)
}

define neg_tail_sup(a: Nat -> Real) -> (Nat -> Real) {
    function(n: Nat) {
        -tail_sup(a, n)
    }
}

define limsup(a: Nat -> Real) -> Real {
    -limit(neg_tail_sup(a))
}

define neg_seq(a: Nat -> Real) -> (Nat -> Real) {
    function(n: Nat) {
        -a(n)
    }
}

define liminf(a: Nat -> Real) -> Real {
    -limsup(neg_seq(a))
}

theorem liminf_add(a: Nat -> Real, b: Nat -> Real) {
    is_bounded_above(a) and is_bounded_below(a) and is_bounded_above(b)
        and is_bounded_below(b)
        implies liminf(a) + liminf(b) <= liminf(add_seq(a, b))
}

theorem liminf_eq_limit(a: Nat -> Real, l: Real) {
    is_bounded_above(a) and is_bounded_below(a) and converges_to(a, l)
        implies liminf(a) = l
}

theorem liminf_eq_limit_tail_inf(a: Nat -> Real) {
    liminf(a) = limit(neg_tail_sup(neg_seq(a)))
}

define tail_inf(a: Nat -> Real, n: Nat) -> Real {
    -tail_sup(neg_seq(a), n)
}

theorem liminf_ge_tail_inf(a: Nat -> Real, n: Nat) {
    is_bounded_above(a) and is_bounded_below(a) implies tail_inf(a, n) <= liminf(a)
}

theorem liminf_le_limsup(a: Nat -> Real) {
    is_bounded_above(a) and is_bounded_below(a) implies liminf(a) <= limsup(a)
}

define one_minus_seq(a: Nat -> Real) -> (Nat -> Real) {
    function(n: Nat) {
        Real.1 - a(n)
    }
}

theorem liminf_one_minus(a: Nat -> Real) {
    is_bounded_above(a) and is_bounded_below(a)
        implies liminf(one_minus_seq(a)) = Real.1 - limsup(a)
}

theorem limsup_add(a: Nat -> Real, b: Nat -> Real) {
    is_bounded_above(a) and is_bounded_below(a) and is_bounded_above(b)
        and is_bounded_below(b)
        implies limsup(add_seq(a, b)) <= limsup(a) + limsup(b)
}

theorem limsup_eq_limit(a: Nat -> Real, l: Real) {
    is_bounded_above(a) and is_bounded_below(a) and converges_to(a, l)
        implies limsup(a) = l
}

theorem limsup_le_tail_sup(a: Nat -> Real, n: Nat) {
    is_bounded_above(a) and is_bounded_below(a) implies limsup(a) <= tail_sup(a, n)
}

theorem limsup_one_minus(a: Nat -> Real) {
    is_bounded_above(a) and is_bounded_below(a)
        implies limsup(one_minus_seq(a)) = Real.1 - liminf(a)
}

numerals Real
theorem local_equiv_data_id[A](s: Set[A]) {
    is_local_equiv_data(s, s, identity_fn[A], identity_fn[A])
}

theorem local_equiv_data_left_inv[A, B](source: Set[A], target: Set[B],
    to_fun: A -> B, inv_fun: B -> A, x: A) {
    is_local_equiv_data(source, target, to_fun, inv_fun) and source.contains(x)
        implies inv_fun(to_fun(x)) = x
}

theorem local_equiv_data_map_source[A, B](source: Set[A], target: Set[B],
    to_fun: A -> B, inv_fun: B -> A, x: A) {
    is_local_equiv_data(source, target, to_fun, inv_fun) and source.contains(x)
        implies target.contains(to_fun(x))
}

theorem local_equiv_data_map_target[A, B](source: Set[A], target: Set[B],
    to_fun: A -> B, inv_fun: B -> A, y: B) {
    is_local_equiv_data(source, target, to_fun, inv_fun) and target.contains(y)
        implies source.contains(inv_fun(y))
}

theorem local_equiv_data_restr[A, B](source: Set[A], target: Set[B],
    to_fun: A -> B, inv_fun: B -> A, s: Set[A]) {
    is_local_equiv_data(source, target, to_fun, inv_fun)
        implies is_local_equiv_data(source.intersection(s),
            target.intersection(set_preimage(inv_fun, s)), to_fun, inv_fun)
}

theorem local_equiv_data_right_inv[A, B](source: Set[A], target: Set[B],
    to_fun: A -> B, inv_fun: B -> A, y: B) {
    is_local_equiv_data(source, target, to_fun, inv_fun) and target.contains(y)
        implies to_fun(inv_fun(y)) = y
}

theorem local_equiv_data_swap[A, B](source: Set[A], target: Set[B],
    to_fun: A -> B, inv_fun: B -> A) {
    is_local_equiv_data(source, target, to_fun, inv_fun)
        implies is_local_equiv_data(target, source, inv_fun, to_fun)
}

theorem local_equiv_data_trans[A, B, C](
    s1: Set[A], t1: Set[B], f1: A -> B, g1: B -> A,
    s2: Set[B], t2: Set[C], f2: B -> C, g2: C -> B) {
    is_local_equiv_data(s1, t1, f1, g1) and is_local_equiv_data(s2, t2, f2, g2)
        implies is_local_equiv_data(s1.intersection(set_preimage(f1, s2)),
            t2.intersection(set_preimage(g2, t1)), compose(f2, f1), compose(g1, g2))
}

let local_equiv_restr[A, B](e: LocalEquiv[A, B], s: Set[A]) -> result: LocalEquiv[A, B] satisfy {
    LocalEquiv[A, B].new(e.source.intersection(s),
        e.target.intersection(set_preimage(e.inv_fun, s)), e.to_fun, e.inv_fun) = Option.some(result)
}

theorem local_equiv_restr_constructible[A, B](e: LocalEquiv[A, B], s: Set[A]) {
    exists(h: LocalEquiv[A, B]) {
        LocalEquiv[A, B].new(e.source.intersection(s),
            e.target.intersection(set_preimage(e.inv_fun, s)), e.to_fun, e.inv_fun) = Option.some(h)
    }
}

theorem local_equiv_restr_inv_fun[A, B](e: LocalEquiv[A, B], s: Set[A]) {
    local_equiv_restr(e, s).inv_fun = e.inv_fun
}

theorem local_equiv_restr_source[A, B](e: LocalEquiv[A, B], s: Set[A]) {
    local_equiv_restr(e, s).source = e.source.intersection(s)
}

theorem local_equiv_restr_target[A, B](e: LocalEquiv[A, B], s: Set[A]) {
    local_equiv_restr(e, s).target = e.target.intersection(set_preimage(e.inv_fun, s))
}

theorem local_equiv_restr_to_fun[A, B](e: LocalEquiv[A, B], s: Set[A]) {
    local_equiv_restr(e, s).to_fun = e.to_fun
}

theorem local_equiv_swap_constructible[A, B](e: LocalEquiv[A, B]) {
    exists(h: LocalEquiv[B, A]) {
        LocalEquiv[B, A].new(e.target, e.source, e.inv_fun, e.to_fun) = Option.some(h)
    }
}

let local_equiv_symm[A, B](e: LocalEquiv[A, B]) -> result: LocalEquiv[B, A] satisfy {
    LocalEquiv[B, A].new(e.target, e.source, e.inv_fun, e.to_fun) = Option.some(result)
}

theorem local_equiv_symm_inv_fun[A, B](e: LocalEquiv[A, B]) {
    local_equiv_symm(e).inv_fun = e.to_fun
}

theorem local_equiv_symm_source[A, B](e: LocalEquiv[A, B]) {
    local_equiv_symm(e).source = e.target
}

theorem local_equiv_symm_target[A, B](e: LocalEquiv[A, B]) {
    local_equiv_symm(e).target = e.source
}

theorem local_equiv_symm_to_fun[A, B](e: LocalEquiv[A, B]) {
    local_equiv_symm(e).to_fun = e.inv_fun
}

let local_equiv_trans[A, B, C](e: LocalEquiv[A, B], e2: LocalEquiv[B, C]) -> result: LocalEquiv[A, C] satisfy {
    LocalEquiv[A, C].new(e.source.intersection(set_preimage(e.to_fun, e2.source)),
        e2.target.intersection(set_preimage(e2.inv_fun, e.target)),
        compose(e2.to_fun, e.to_fun), compose(e.inv_fun, e2.inv_fun)) = Option.some(result)
}

theorem local_equiv_trans_inv_fun[A, B, C](e: LocalEquiv[A, B], e2: LocalEquiv[B, C]) {
    local_equiv_trans(e, e2).inv_fun = compose(e.inv_fun, e2.inv_fun)
}

theorem local_equiv_trans_source[A, B, C](e: LocalEquiv[A, B], e2: LocalEquiv[B, C]) {
    local_equiv_trans(e, e2).source = e.source.intersection(set_preimage(e.to_fun, e2.source))
}

theorem local_equiv_trans_target[A, B, C](e: LocalEquiv[A, B], e2: LocalEquiv[B, C]) {
    local_equiv_trans(e, e2).target = e2.target.intersection(set_preimage(e2.inv_fun, e.target))
}

theorem local_equiv_trans_to_fun[A, B, C](e: LocalEquiv[A, B], e2: LocalEquiv[B, C]) {
    local_equiv_trans(e, e2).to_fun = compose(e2.to_fun, e.to_fun)
}

numerals Nat
theorem lonely_runner_two(v: Real) {
    Real.1 <= v implies exists(t: Real) {
        Real.1 / from_nat[Real](Nat.3) <= circular_distance(v * t)
    }
}

theorem lt_nat_multiple(x: Real, eps: Real) {
    eps > Real.0 implies exists(n: Nat) {
        x < eps * from_nat[Real](n)
    }
}

theorem lte_of_lte_add_eps(x: Real, y: Real) {
    (forall(eps: Real) { eps.is_positive implies x <= y + eps }) implies x <= y
}

numerals Real
theorem metric_tail_bound_at[M: MetricSpace](q: Nat -> M, a: M, n: Nat, eps: Real, i: Nat) {
    metric_tail_bound(q, a, n, eps) and n <= i implies q(i).distance(a) < eps
}

theorem metric_tail_bound_from_distances[M: MetricSpace](q: Nat -> M, a: M, n: Nat, eps: Real) {
    (forall(i: Nat) {
        n <= i implies q(i).distance(a) < eps
    }) implies metric_tail_bound(q, a, n, eps)
}

numerals Nat
theorem nat_lt_one_four {
    Nat.1 < Nat.4
}

theorem nat_lt_one_three {
    Nat.1 < Nat.3
}

theorem nat_lt_one_two {
    Nat.1 < Nat.2
}

theorem nat_lt_two_four {
    Nat.2 < Nat.4
}

theorem nat_one_le_one {
    Nat.1 <= Nat.1
}

theorem nat_one_le_three {
    Nat.1 <= Nat.3
}

theorem nat_one_le_two {
    Nat.1 <= Nat.2
}

numerals Real
theorem nat_zero_lte(n: Nat) {
    Nat.0 <= n
}

theorem neg_reverses_lte(x: Real, y: Real) {
    x <= y implies -y <= -x
}

theorem neg_seq_bounded_above(a: Nat -> Real) {
    is_bounded_below(a) implies is_bounded_above(neg_seq(a))
}

theorem neg_seq_bounded_below(a: Nat -> Real) {
    is_bounded_above(a) implies is_bounded_below(neg_seq(a))
}

theorem neg_tail_sup_converges(a: Nat -> Real) {
    is_bounded_above(a) and is_bounded_below(a) implies converges(neg_tail_sup(a))
}

theorem neg_tail_sup_neg_seq_eq_tail_inf(a: Nat -> Real, k: Nat) {
    neg_tail_sup(neg_seq(a))(k) = tail_inf(a, k)
}

numerals Nat
theorem one_minus_frac(k: Nat, n: Nat) {
    k < n implies Real.1 - from_nat[Real](k) / from_nat[Real](n) = from_nat[Real](n - k) / from_nat[Real](n)
}

theorem one_minus_one_minus(x: Real) {
    Real.1 - (Real.1 - x) = x
}

theorem open_big_union[T: TopologicalSpace](c: Set[T] -> Bool) {
    (forall(s: Set[T]) { c(s) implies T.is_open(s) })
        implies T.is_open(big_union(c))
}

theorem open_empty[T: TopologicalSpace] {
    T.is_open(Set[T].empty_set)
}

theorem open_iff_neighborhood_of_each_point[T: TopologicalSpace](s: Set[T]) {
    T.is_open(s) = forall(x: T) {
        s.contains(x) implies is_neighborhood(s, x)
    }
}

theorem open_inter[T: TopologicalSpace](s: Set[T], t: Set[T]) {
    T.is_open(s) and T.is_open(t) implies T.is_open(s.intersection(t))
}

theorem open_universal[T: TopologicalSpace] {
    T.is_open(Set[T].universal_set)
}

define prefix_member(c: Set[Real] -> Bool, lower: Real, upper: Real, x: Real) -> Bool {
    lower <= x and x <= upper and finitely_coverable(c, lower, x)
}

define prefix_set(c: Set[Real] -> Bool, lower: Real, upper: Real) -> Set[Real] {
    Set[Real].new(prefix_member(c, lower, upper))
}

theorem product_open_empty[X: TopologicalSpace, Y: TopologicalSpace] {
    is_open_in_product[X, Y](Set[Pair[X, Y]].empty_set)
}

theorem product_open_universal[X: TopologicalSpace, Y: TopologicalSpace] {
    is_open_in_product[X, Y](Set[Pair[X, Y]].universal_set)
}

numerals Real
theorem real_converges_to_imp_tendsto_metric(q: Nat -> Real, a: Real) {
    converges_to(q, a) implies tendsto_metric(q, a)
}

numerals Int
theorem real_from_int_lte(m: Int, n: Int) {
    m <= n implies Real.from_int(m) <= Real.from_int(n)
}

numerals Real
theorem real_is_cauchy_metric_imp_is_cauchy_seq(q: Nat -> Real) {
    is_cauchy_metric(q) implies is_cauchy_seq(q)
}

numerals Real
theorem real_norm_eq_abs(x: Real) {
    real_norm(x) = x.abs
}

numerals Nat
theorem same_denom_lte(a: Nat, b: Nat, d: Nat) {
    Nat.0 < d and a <= b
    implies from_nat[Real](a) / from_nat[Real](d) <= from_nat[Real](b) / from_nat[Real](d)
}

numerals Nat
theorem tail_inf_bounds_terms(a: Nat -> Real, n: Nat, k: Nat) {
    is_bounded_below(a) and n <= k implies tail_inf(a, n) <= a(k)
}

theorem tail_inf_is_greatest(a: Nat -> Real, n: Nat, b: Real) {
    is_bounded_below(a) and (forall(k: Nat) { n <= k implies b <= a(k) })
        implies b <= tail_inf(a, n)
}

theorem tail_sup_bounds_terms(a: Nat -> Real, n: Nat, k: Nat) {
    is_bounded_above(a) and n <= k implies a(k) <= tail_sup(a, n)
}

theorem tail_sup_is_least(a: Nat -> Real, n: Nat, b: Real) {
    is_bounded_above(a) and (forall(k: Nat) { n <= k implies a(k) <= b })
        implies tail_sup(a, n) <= b
}

numerals Real
theorem tendsto_metric_tail_bound[M: MetricSpace](q: Nat -> M, a: M, eps: Real) {
    tendsto_metric(q, a) and eps.is_positive implies exists(n: Nat) {
        metric_tail_bound(q, a, n, eps)
    }
}

theorem topological_open_eq_metric_open[M: MetricTopologicalSpace](s: Set[M]) {
    M.is_open(s) = is_open_metric(s)
}


/// A Cauchy sequence has bounded tails.
theorem is_cauchy_metric_bound[M: MetricSpace](q: Nat -> M, eps: Real) {
    is_cauchy_metric(q) and eps.is_positive implies exists(n: Nat) {
        cauchy_metric_bound(q, n, eps)
    }
}

/// A set is metric-closed when its complement is metric-open.
define is_closed_metric[M: MetricSpace](s: Set[M]) -> Bool {
    is_open_metric(s.c)
}

/// Points outside a closed ball have an open neighborhood outside the ball.
theorem closed_ball_complement_has_open_neighborhood[M: MetricSpace](xc: M, rad: Real, y: M) {
    closed_ball(xc, rad).c.contains(y) implies exists(r: Real) {
        r.is_positive and open_ball(y, r).subset(closed_ball(xc, rad).c)
    }
}

/// A sequence tends to a point when every positive radius has a bounded tail.
theorem tendsto_metric_from_tail_bounds[M: MetricSpace](q: Nat -> M, a: M) {
    (forall(eps: Real) {
        eps.is_positive implies exists(n: Nat) {
            metric_tail_bound(q, a, n, eps)
        }
    }) implies tendsto_metric(q, a)
}

/// A sequence is Cauchy when every positive radius has a pairwise bounded tail.
theorem is_cauchy_metric_from_bounds[M: MetricSpace](q: Nat -> M) {
    (forall(eps: Real) {
        eps.is_positive implies exists(n: Nat) {
            cauchy_metric_bound(q, n, eps)
        }
    }) implies is_cauchy_metric(q)
}

/// A map has the Lipschitz bound when the distance inequality holds for every pair.
theorem lipschitz_bound_from_distances[M: MetricSpace, N: MetricSpace](k: Real, f: M -> N) {
    (forall(x: M, y: M) {
        f(x).distance(f(y)) <= k * x.distance(y)
    }) implies lipschitz_bound(k, f)
}

/// An isometry preserves the tail-bound form of metric Cauchy sequences.
theorem isometry_preserves_cauchy_metric_bounds[M: MetricSpace, N: MetricSpace](f: M -> N, q: Nat -> M) {
    is_isometry(f) and is_cauchy_metric(q) implies forall(eps: Real) {
        eps.is_positive implies exists(n: Nat) {
            cauchy_metric_bound(compose(f, q), n, eps)
        }
    }
}

/// Boundedness above yields a bound.
theorem is_bounded_above_witness(a: Nat -> Real) {
    is_bounded_above(a) implies exists(b: Real) { is_upper_bound(a, b) }
}

/// Membership in a big union is membership in some set of the family.
theorem big_union_contains_eq[T](c: Set[T] -> Bool, x: T) {
    big_union(c).contains(x) = exists(s: Set[T]) {
        c(s) and s.contains(x)
    }
}

/// A ball around `m` covers the interval `[s, m]` when `s` lies within the ball and below `m`.
theorem interval_after_ball_subset(m: Real, eps: Real, s: Real, u: Set[Real]) {
    m - eps < s and s <= m and eps.is_positive
        and (forall(w: Real) { w.is_close(m, eps) implies u.contains(w) })
    implies closed_interval_set(s, m).subset(u)
}

/// Boundedness below yields a bound.
theorem is_bounded_below_witness(a: Nat -> Real) {
    is_bounded_below(a) implies exists(b: Real) { is_lower_bound(a, b) }
}

/// An isometry preserves the tail-bound form of metric convergence.
theorem isometry_preserves_tendsto_metric_bounds[M: MetricSpace, N: MetricSpace](f: M -> N, q: Nat -> M, a: M) {
    is_isometry(f) and tendsto_metric(q, a) implies forall(eps: Real) {
        eps.is_positive implies exists(n: Nat) {
            metric_tail_bound(compose(f, q), f(a), n, eps)
        }
    }
}

/// A ball around `m` covers the interval `[m, y]` when `y` lies within the ball and above `m`.
theorem interval_before_ball_subset(m: Real, eps: Real, y: Real, u: Set[Real]) {
    y < m + eps and m <= y and eps.is_positive
        and (forall(w: Real) { w.is_close(m, eps) implies u.contains(w) })
    implies closed_interval_set(m, y).subset(u)
}

/// Membership in the interior is membership in an open subset of the set.
theorem interior_contains_eq[T: TopologicalSpace](s: Set[T], x: T) {
    interior(s).contains(x) = exists(u: Set[T]) {
        T.is_open(u) and u.subset(s) and u.contains(x)
    }
}

/// Membership in a prefix set is the defining interval and finite-cover condition.
theorem prefix_set_contains_eq(c: Set[Real] -> Bool, lower: Real, upper: Real, x: Real) {
    prefix_set(c, lower, upper).contains(x) =
        (lower <= x and x <= upper and finitely_coverable(c, lower, x))
}

/// Membership in the tail set is being a value from the index onward.
theorem tail_set_contains_eq(a: Nat -> Real, n: Nat, x: Real) {
    tail_set(a, n).contains(x) = tail_value(a, n, x)
}

/// An open set makes each of its points an interior point.
theorem real_open_is_interior_point(s: Set[Real], x: Real) {
    real_is_open(s) and s.contains(x) implies real_is_interior_point(s, x)
}

/// Interior membership is equivalent to being a neighborhood point.
theorem interior_iff_neighborhood[T: TopologicalSpace](s: Set[T], x: T) {
    interior(s).contains(x) = is_neighborhood(s, x)
}

/// Open rectangles are basis boxes.
theorem box_is_box[X: TopologicalSpace, Y: TopologicalSpace](u: Set[X], v: Set[Y]) {
    X.is_open(u) and Y.is_open(v) implies is_box[X, Y](box(u, v))
}

/// Intersecting boxes containing a point produces a box containing that point.
theorem box_inter_is_box_with_point[X: TopologicalSpace, Y: TopologicalSpace](
    b1: Set[Pair[X, Y]], b2: Set[Pair[X, Y]], p: Pair[X, Y]) {
    is_box[X, Y](b1) and is_box[X, Y](b2) and b1.contains(p) and b2.contains(p)
        implies is_box[X, Y](b1.intersection(b2)) and b1.intersection(b2).contains(p)
}
