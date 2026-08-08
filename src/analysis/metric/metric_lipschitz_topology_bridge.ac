from data.basic.set import Set, set_preimage, set_preimage_contains_eq, subset_contains
from analysis.metric.metric_space import MetricSpace, is_open_metric, open_ball,
    is_lipschitz_with, is_lipschitz, is_isometry,
    lipschitz_with_constant_non_negative, lipschitz_with_distance,
    isometry_is_lipschitz_with_one
from analysis.metric.metric_topology import MetricTopologicalSpace, topological_open_is_metric_open,
    metric_open_is_topological_open
from analysis.topology.topological_space import is_continuous
from real import Real, exists_small_mul, lte_mul_nonneg_left
from order import lte_trans, lte_lt_trans
from data.basic.logic import by_contradiction, not_forall_imp_exists_not
numerals Real

/// A Lipschitz map pulls metric-open target sets back to metric-open source sets.
theorem lipschitz_with_preimage_metric_open[M: MetricSpace, N: MetricSpace](
    k: Real, f: M -> N, v: Set[N]
) {
    is_lipschitz_with(k, f) and is_open_metric(v) implies is_open_metric(set_preimage(f, v))
} by {
    if is_lipschitz_with(k, f) and is_open_metric(v) {
        lipschitz_with_constant_non_negative(k, f)
        not k.is_negative
        let u: Set[M] = set_preimage(f, v)
        forall(x: M) {
            if u.contains(x) {
                set_preimage_contains_eq(f, v, x)
                v.contains(f(x))
                is_open_metric(v) = forall(z: N) {
                    v.contains(z) implies exists(r: Real) {
                        r.is_positive and open_ball(z, r).subset(v)
                    }
                }
                let eps: Real satisfy {
                    eps.is_positive and open_ball(f(x), eps).subset(v)
                }
                exists_small_mul(k, eps)
                let delta: Real satisfy {
                    delta.is_positive and k * delta < eps
                }
                forall(y: M) {
                    if open_ball(x, delta).contains(y) {
                        open_ball(x, delta).contains(y) = (y.distance(x) < delta)
                        y.distance(x) < delta
                        y.distance(x) <= delta
                        lipschitz_with_distance(k, f, y, x)
                        f(y).distance(f(x)) <= k * y.distance(x)
                        lte_mul_nonneg_left(y.distance(x), delta, k)
                        k * y.distance(x) <= k * delta
                        lte_trans(f(y).distance(f(x)), k * y.distance(x), k * delta)
                        f(y).distance(f(x)) <= k * delta
                        lte_lt_trans(f(y).distance(f(x)), k * delta, eps)
                        f(y).distance(f(x)) < eps
                        open_ball(f(x), eps).contains(f(y))
                        subset_contains(open_ball(f(x), eps), v, f(y))
                        v.contains(f(y))
                        set_preimage_contains_eq(f, v, y)
                        u.contains(y)
                    }
                }
                open_ball(x, delta).subset(u)
                exists(r: Real) {
                    r.is_positive and open_ball(x, r).subset(u)
                }
            }
        }
        is_open_metric(u) = forall(x: M) {
            u.contains(x) implies exists(r: Real) {
                r.is_positive and open_ball(x, r).subset(u)
            }
        }
        if not is_open_metric(u) {
            not_forall_imp_exists_not(function(x: M) {
                u.contains(x) implies exists(r: Real) {
                    r.is_positive and open_ball(x, r).subset(u)
                }
            })
            let w: M satisfy {
                not (u.contains(w) implies exists(r: Real) {
                    r.is_positive and open_ball(w, r).subset(u)
                })
            }
            u.contains(w) implies exists(r: Real) {
                r.is_positive and open_ball(w, r).subset(u)
            }
            false
        }
        by_contradiction(is_open_metric(u))
        is_open_metric(u)
        is_open_metric(set_preimage(f, v))
    }
}

/// A Lipschitz map pulls metric-open target sets back to metric-open source sets.
theorem lipschitz_preimage_metric_open[M: MetricSpace, N: MetricSpace](
    f: M -> N, v: Set[N]
) {
    is_lipschitz(f) and is_open_metric(v) implies is_open_metric(set_preimage(f, v))
} by {
    if is_lipschitz(f) and is_open_metric(v) {
        let k: Real satisfy {
            is_lipschitz_with(k, f)
        }
        lipschitz_with_preimage_metric_open(k, f, v)
        is_open_metric(set_preimage(f, v))
    }
}

/// A Lipschitz map between compatible metric topologies is continuous.
theorem lipschitz_with_is_continuous[M: MetricTopologicalSpace, N: MetricTopologicalSpace](
    k: Real, f: M -> N
) {
    is_lipschitz_with(k, f) implies is_continuous(f)
} by {
    if is_lipschitz_with(k, f) {
        forall(v: Set[N]) {
            if N.is_open(v) {
                topological_open_is_metric_open(v)
                is_open_metric(v)
                lipschitz_with_preimage_metric_open(k, f, v)
                is_open_metric(set_preimage(f, v))
                metric_open_is_topological_open(set_preimage(f, v))
                M.is_open(set_preimage(f, v))
            }
        }
        is_continuous(f) = forall(v: Set[N]) {
            N.is_open(v) implies M.is_open(set_preimage(f, v))
        }
        is_continuous(f)
    }
}

/// A Lipschitz map between compatible metric topologies is continuous.
theorem lipschitz_is_continuous[M: MetricTopologicalSpace, N: MetricTopologicalSpace](
    f: M -> N
) {
    is_lipschitz(f) implies is_continuous(f)
} by {
    if is_lipschitz(f) {
        let k: Real satisfy {
            is_lipschitz_with(k, f)
        }
        lipschitz_with_is_continuous(k, f)
        is_continuous(f)
    }
}

/// An isometry between compatible metric topologies is continuous.
theorem isometry_is_continuous[M: MetricTopologicalSpace, N: MetricTopologicalSpace](
    f: M -> N
) {
    is_isometry(f) implies is_continuous(f)
} by {
    if is_isometry(f) {
        isometry_is_lipschitz_with_one(f)
        is_lipschitz_with(1, f)
        lipschitz_with_is_continuous(1, f)
        is_continuous(f)
    }
}
