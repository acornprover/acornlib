/// Sequence facts in metric spaces.
///
/// Convergent and Cauchy sequences are bounded, subsequences of convergent
/// sequences converge to the same limit, and limits of sequences in closed
/// sets stay in the closed set.  The closed-set fact specializes to closed
/// intervals of the real line.

from nat import Nat, lt_suc, lt_imp_lte_suc
from order import is_monotone, monotone_apply, lt_imp_lte, lte_trans, lte_ref,
    lte_antisymm, not_lte_imp_gt, not_lt_self, lte_lt_trans, lt_trans,
    closed_interval
from real import Real, is_unbounded, lt_imp_minus_pos, close_imp_bounds,
    tail_bound, tail_bound_implies_is_close, converges_to, lte_abs, abs_gte_zero,
    neg_distrib, neg_neg, neg_zero, add_neg_eq_zero, add_zero_left, add_zero_right,
    add_assoc, add_comm
from order_set import closed_interval_set, closed_interval_set_contains_eq,
    closed_interval_set_lower_le, closed_interval_set_le_upper
from algebra.add_ordered_group import add_le_add, add_lt_add_left, add_lt_add_right
from data.basic.set import Set, subset_contains, subset_contains_eq, compl_contains_eq
from data.basic.functions import compose
from data.basic.logic import by_contradiction, not_forall_imp_exists_not, not_implies
from analysis.metric.metric_space import MetricSpace, distance_self, is_bounded, closed_ball,
    closed_ball_contains_iff, open_ball, open_ball_contains_iff, is_closed_metric,
    is_open_metric, tendsto_metric, metric_tail_bound, metric_tail_bound_at,
    metric_tail_bound_from_distances, tendsto_metric_tail_bound,
    tendsto_metric_from_tail_bounds, tendsto_metric_imp_cauchy, is_cauchy_metric,
    cauchy_metric_bound, cauchy_metric_bound_at, is_cauchy_metric_bound
from analysis.metric.complete_metric_space import CompleteMetricSpace,
    real_tendsto_metric_imp_converges_to

numerals Real

/// Zero is at most every natural number.
theorem nat_zero_lte(n: Nat) {
    Nat.0 <= n
} by {
    define p(k: Nat) -> Bool {
        Nat.0 <= k
    }
    lte_ref(Nat.0)
    p(Nat.0)
    forall(k: Nat) {
        if p(k) {
            lt_suc(k)
            k < k.suc
            lt_imp_lte(k, k.suc)
            k <= k.suc
            lte_trans(Nat.0, k, k.suc)
            Nat.0 <= k.suc
            p(k.suc)
        }
    }
    p(Nat.0) and forall(k: Nat) { p(k) implies p(k.suc) }
    Nat.induction(p)
    p(n)
    Nat.0 <= n
}

/// A real at least both of two given reals.
theorem real_lte_upper_bound_pair(a: Real, b: Real) {
    exists(c: Real) {
        a <= c and b <= c
    }
} by {
    let c: Real = a.abs + b.abs + Real.1
    lte_abs(a)
    a <= a.abs
    lte_abs(b)
    b <= b.abs
    abs_gte_zero(a)
    Real.0 <= a.abs
    abs_gte_zero(b)
    Real.0 <= b.abs
    lte_ref(a.abs)
    add_le_add(a.abs, a.abs, Real.0, b.abs)
    a.abs + Real.0 <= a.abs + b.abs
    a.abs + Real.0 = a.abs
    a.abs <= a.abs + b.abs
    lte_trans(a, a.abs, a.abs + b.abs)
    a <= a.abs + b.abs
    lte_ref(b.abs)
    add_le_add(b.abs, b.abs, Real.0, a.abs)
    b.abs + Real.0 <= b.abs + a.abs
    b.abs + Real.0 = b.abs
    b.abs <= b.abs + a.abs
    b.abs + a.abs = a.abs + b.abs
    lte_trans(b, b.abs, a.abs + b.abs)
    b <= a.abs + b.abs
    Real.1.is_positive
    add_lt_add_left(Real.0, Real.1, a.abs + b.abs)
    (a.abs + b.abs) + Real.0 < (a.abs + b.abs) + Real.1
    (a.abs + b.abs) + Real.0 = a.abs + b.abs
    a.abs + b.abs < c
    lte_lt_trans(a, a.abs + b.abs, c)
    a < c
    lt_imp_lte(a, c)
    a <= c
    lte_lt_trans(b, a.abs + b.abs, c)
    b < c
    lt_imp_lte(b, c)
    b <= c
    a <= c and b <= c
    exists(d: Real) { a <= d and b <= d }
}

/// Subtraction cancels through a common minuend: (a - b) - (a - c) = c - b.
theorem sub_sub_cancel(a: Real, b: Real, c: Real) {
    (a - b) - (a - c) = c - b
} by {
    neg_distrib(a, -c)
    -(a + -c) = -a + -(-c)
    neg_neg(c)
    -(-c) = c
    -a + -(-c) = -a + c
    a - c = a + -c
    -(a - c) = -(a + -c)
    (a - b) - (a - c) = (a + -b) + -(a - c)
    (a + -b) + -(a - c) = (a + -b) + (-a + c)
    add_assoc(a, -b, -a + c)
    (a + -b) + (-a + c) = a + (-b + (-a + c))
    add_assoc(-b, -a, c)
    -b + (-a + c) = (-b + -a) + c
    add_comm(-b, -a)
    -b + -a = -a + -b
    (-b + -a) + c = (-a + -b) + c
    a + (-b + (-a + c)) = a + ((-a + -b) + c)
    add_assoc(a, -a + -b, c)
    a + ((-a + -b) + c) = (a + (-a + -b)) + c
    add_assoc(a, -a, -b)
    (a + -a) + -b = a + (-a + -b)
    add_neg_eq_zero(a)
    a + -a = Real.0
    ((a + -a) + -b) + c = (Real.0 + -b) + c
    add_zero_left(-b)
    Real.0 + -b = -b
    (Real.0 + -b) + c = -b + c
    c - b = c + -b
    (a - b) - (a - c) = c - b
}

/// Subtracting a difference cancels: a - (a - b) = b.
theorem sub_minus_sub(a: Real, b: Real) {
    a - (a - b) = b
} by {
    sub_sub_cancel(a, Real.0, b)
    (a - Real.0) - (a - b) = b - Real.0
    neg_zero
    -Real.0 = Real.0
    a - Real.0 = a + -Real.0
    a + -Real.0 = a + Real.0
    add_zero_right(a)
    a + Real.0 = a
    a - Real.0 = a
    b - Real.0 = b + -Real.0
    b + -Real.0 = b + Real.0
    add_zero_right(b)
    b + Real.0 = b
    b - Real.0 = b
    a - (a - b) = b
}

/// Adding a difference cancels: a + (b - a) = b.
theorem add_sub_cancel(a: Real, b: Real) {
    a + (b - a) = b
} by {
    a + (b - a) = a + (b + -a)
    add_assoc(a, b, -a)
    (a + b) + -a = a + (b + -a)
    add_comm(a, b)
    a + b = b + a
    (a + b) + -a = (b + a) + -a
    add_assoc(b, a, -a)
    (b + a) + -a = b + (a + -a)
    add_neg_eq_zero(a)
    a + -a = Real.0
    b + (a + -a) = b + Real.0
    add_zero_right(b)
    b + Real.0 = b
    a + (b - a) = b
}

/// True if `x` is a value of the sequence `q`.
define sequence_range_contains[M](q: Nat -> M, x: M) -> Bool {
    exists(n: Nat) { q(n) = x }
}

/// The set of values taken by a sequence.
define sequence_range[M](q: Nat -> M) -> Set[M] {
    Set[M].new(sequence_range_contains(q))
}

/// Membership in a sequence range is equivalent to being a value of the
/// sequence.
theorem sequence_range_contains_eq[M](q: Nat -> M, x: M) {
    sequence_range(q).contains(x) = sequence_range_contains(q, x)
}

/// Membership in a sequence range is membership in its value predicate.
theorem sequence_range_contains_iff[M](q: Nat -> M, x: M) {
    sequence_range_contains(q, x) = exists(n: Nat) { q(n) = x }
}

/// A subsequence of a convergent metric sequence converges to the same limit.
///
/// The index map is a nondecreasing unbounded map `f`; every tail of `q`
/// lies within `eps` of `a`, and since `f` is unbounded some tail of the
/// indices is mapped beyond the tail threshold.
theorem metric_subsequence_tendsto[M: MetricSpace](q: Nat -> M, f: Nat -> Nat, a: M) {
    is_monotone(f) and is_unbounded(f) and tendsto_metric(q, a)
    implies tendsto_metric(compose(q, f), a)
} by {
    if is_monotone(f) and is_unbounded(f) and tendsto_metric(q, a) {
        forall(eps: Real) {
            if eps.is_positive {
                tendsto_metric_tail_bound(q, a, eps)
                let n: Nat satisfy {
                    metric_tail_bound(q, a, n, eps)
                }
                is_unbounded(f) = forall(bound: Nat) {
                    exists(k: Nat) { bound < f(k) }
                }
                exists(k: Nat) { n < f(k) }
                let k: Nat satisfy {
                    n < f(k)
                }
                forall(i: Nat) {
                    if k <= i {
                        monotone_apply(f, k, i)
                        f(k) <= f(i)
                        lt_imp_lte(n, f(k))
                        n <= f(k)
                        lte_trans(n, f(k), f(i))
                        n <= f(i)
                        metric_tail_bound_at(q, a, n, eps, f(i))
                        q(f(i)).distance(a) < eps
                        compose(q, f)(i).distance(a) < eps
                    }
                }
                metric_tail_bound_from_distances(compose(q, f), a, k, eps)
                metric_tail_bound(compose(q, f), a, k, eps)
                exists(m: Nat) {
                    metric_tail_bound(compose(q, f), a, m, eps)
                }
            }
        }
        if not tendsto_metric(compose(q, f), a) {
            tendsto_metric(compose(q, f), a) = forall(e: Real) {
                e.is_positive implies exists(n: Nat) {
                    metric_tail_bound(compose(q, f), a, n, e)
                }
            }
            not forall(e: Real) {
                e.is_positive implies exists(n: Nat) {
                    metric_tail_bound(compose(q, f), a, n, e)
                }
            }
            not_forall_imp_exists_not[Real](function(e: Real) {
                e.is_positive implies exists(n: Nat) {
                    metric_tail_bound(compose(q, f), a, n, e)
                }
            })
            let e: Real satisfy {
                not (e.is_positive implies exists(n: Nat) {
                    metric_tail_bound(compose(q, f), a, n, e)
                })
            }
            not_implies(e.is_positive, exists(n: Nat) {
                metric_tail_bound(compose(q, f), a, n, e)
            })
            (not (e.is_positive implies exists(n: Nat) {
                metric_tail_bound(compose(q, f), a, n, e)
            })) = (e.is_positive and not exists(n: Nat) {
                metric_tail_bound(compose(q, f), a, n, e)
            })
            e.is_positive and not exists(n: Nat) {
                metric_tail_bound(compose(q, f), a, n, e)
            }
            e.is_positive
            forall(e2: Real) {
                e2.is_positive implies exists(m: Nat) {
                    metric_tail_bound(compose(q, f), a, m, e2)
                }
            }
            e.is_positive implies exists(m: Nat) {
                metric_tail_bound(compose(q, f), a, m, e)
            }
            exists(m: Nat) {
                metric_tail_bound(compose(q, f), a, m, e)
            }
            not exists(n: Nat) {
                metric_tail_bound(compose(q, f), a, n, e)
            }
            false
        }
        by_contradiction(tendsto_metric(compose(q, f), a))
        tendsto_metric(compose(q, f), a)
    }
}

/// A Cauchy sequence in a metric space is bounded.
///
/// The proof bounds the finite prefix up to the Cauchy threshold by
/// induction, and the infinite tail by the triangle inequality against the
/// threshold term.
theorem cauchy_metric_seq_is_bounded[M: MetricSpace](q: Nat -> M) {
    is_cauchy_metric(q) implies is_bounded(sequence_range(q))
} by {
    if is_cauchy_metric(q) {
        Real.1.is_positive
        is_cauchy_metric_bound(q, Real.1)
        let n0: Nat satisfy {
            cauchy_metric_bound(q, n0, Real.1)
        }
        define p(k: Nat) -> Bool {
            exists(r: Real) {
                forall(i: Nat) {
                    i <= k implies q(i).distance(q(Nat.0)) <= r
                }
            }
        }
        // base case: radius zero covers the first term
        forall(i: Nat) {
            if i <= Nat.0 {
                i = Nat.0
                distance_self(q(Nat.0))
                q(Nat.0).distance(q(Nat.0)) = Real.0
                q(i).distance(q(Nat.0)) = Real.0
                q(i).distance(q(Nat.0)) <= Real.0
            }
        }
        p(Nat.0)
        // step: a radius covering the prefix also covers the next term when
        // enlarged to a common upper bound of the old radius and the new
        // distance
        forall(k: Nat) {
            if p(k) {
                let r: Real satisfy {
                    forall(i: Nat) {
                        i <= k implies q(i).distance(q(Nat.0)) <= r
                    }
                }
                real_lte_upper_bound_pair(r, q(k.suc).distance(q(Nat.0)))
                let c: Real satisfy {
                    r <= c and q(k.suc).distance(q(Nat.0)) <= c
                }
                forall(i: Nat) {
                    if i <= k.suc {
                        if i <= k {
                            q(i).distance(q(Nat.0)) <= r
                            lte_trans(q(i).distance(q(Nat.0)), r, c)
                            q(i).distance(q(Nat.0)) <= c
                        } else {
                            not i <= k
                            not_lte_imp_gt(i, k)
                            i > k
                            k < i
                            lt_imp_lte_suc(k, i)
                            k.suc <= i
                            i <= k.suc
                            lte_antisymm(k.suc, i)
                            k.suc = i
                            q(k.suc).distance(q(Nat.0)) <= c
                            q(i).distance(q(Nat.0)) <= c
                        }
                    }
                }
                p(k.suc)
            }
        }
        p(Nat.0) and forall(k: Nat) { p(k) implies p(k.suc) }
        Nat.induction(p)
        p(n0)
        let r0: Real satisfy {
            forall(i: Nat) {
                i <= n0 implies q(i).distance(q(Nat.0)) <= r0
            }
        }
        // tail terms beyond the Cauchy threshold are within 1 + d(q(n0), q(0))
        // of q(0)
        let tail_cap: Real = Real.1 + q(n0).distance(q(Nat.0))
        forall(i: Nat) {
            if n0 <= i {
                cauchy_metric_bound_at(q, n0, Real.1, i, n0)
                q(i).distance(q(n0)) < Real.1
                q(i).distance(q(Nat.0)) <= q(i).distance(q(n0)) + q(n0).distance(q(Nat.0))
                add_lt_add_right(q(i).distance(q(n0)), Real.1, q(n0).distance(q(Nat.0)))
                q(i).distance(q(n0)) + q(n0).distance(q(Nat.0)) < tail_cap
                lte_lt_trans(q(i).distance(q(Nat.0)),
                    q(i).distance(q(n0)) + q(n0).distance(q(Nat.0)), tail_cap)
                q(i).distance(q(Nat.0)) < tail_cap
                lt_imp_lte(q(i).distance(q(Nat.0)), tail_cap)
                q(i).distance(q(Nat.0)) <= tail_cap
            }
        }
        // one radius covers both the prefix bound and the tail cap
        real_lte_upper_bound_pair(r0, tail_cap)
        let rr: Real satisfy {
            r0 <= rr and tail_cap <= rr
        }
        forall(x: M) {
            if sequence_range(q).contains(x) {
                sequence_range_contains_eq(q, x)
                sequence_range_contains(q, x)
                sequence_range_contains_iff(q, x)
                exists(n: Nat) { q(n) = x }
                let n: Nat satisfy {
                    q(n) = x
                }
                if n <= n0 {
                    q(n).distance(q(Nat.0)) <= r0
                    lte_trans(q(n).distance(q(Nat.0)), r0, rr)
                    q(n).distance(q(Nat.0)) <= rr
                } else {
                    not n <= n0
                    not_lte_imp_gt(n, n0)
                    n > n0
                    n0 < n
                    lt_imp_lte(n0, n)
                    n0 <= n
                    q(n).distance(q(Nat.0)) <= tail_cap
                    lte_trans(q(n).distance(q(Nat.0)), tail_cap, rr)
                    q(n).distance(q(Nat.0)) <= rr
                }
                closed_ball_contains_iff(q(Nat.0), rr, q(n))
                closed_ball(q(Nat.0), rr).contains(q(n)) = (q(n).distance(q(Nat.0)) <= rr)
                closed_ball(q(Nat.0), rr).contains(q(n))
                q(n) = x
                closed_ball(q(Nat.0), rr).contains(x)
            }
        }
        subset_contains_eq(sequence_range(q), closed_ball(q(Nat.0), rr))
        sequence_range(q).subset(closed_ball(q(Nat.0), rr))
        exists(c: M, r: Real) {
            sequence_range(q).subset(closed_ball(c, r))
        }
        is_bounded(sequence_range(q)) = exists(c: M, r: Real) {
            sequence_range(q).subset(closed_ball(c, r))
        }
        is_bounded(sequence_range(q))
    }
}

/// A convergent sequence in a metric space is bounded.
theorem tendsto_metric_seq_is_bounded[M: MetricSpace](q: Nat -> M, a: M) {
    tendsto_metric(q, a) implies is_bounded(sequence_range(q))
} by {
    if tendsto_metric(q, a) {
        tendsto_metric_imp_cauchy(q, a)
        is_cauchy_metric(q)
        cauchy_metric_seq_is_bounded(q)
        is_bounded(sequence_range(q))
    }
}

/// The limit of a sequence in a closed set stays in the closed set.
///
/// If the limit lay outside the closed set, its complement would be open and
/// contain a ball around the limit disjoint from the set, but the tail of the
/// sequence lies in that ball, contradicting that the sequence stays in the
/// set.
theorem tendsto_metric_in_closed_set[M: MetricSpace](s: Set[M], q: Nat -> M, a: M) {
    is_closed_metric(s) and (forall(n: Nat) { s.contains(q(n)) }) and tendsto_metric(q, a)
    implies s.contains(a)
} by {
    if is_closed_metric(s) and (forall(n: Nat) { s.contains(q(n)) }) and tendsto_metric(q, a) {
        if not s.contains(a) {
            compl_contains_eq(s, a)
            s.c.contains(a) = not s.contains(a)
            s.c.contains(a)
            is_closed_metric(s) = is_open_metric(s.c)
            is_open_metric(s.c) = forall(x: M) {
                s.c.contains(x) implies exists(r: Real) {
                    r.is_positive and open_ball(x, r).subset(s.c)
                }
            }
            exists(r: Real) {
                r.is_positive and open_ball(a, r).subset(s.c)
            }
            let r: Real satisfy {
                r.is_positive and open_ball(a, r).subset(s.c)
            }
            tendsto_metric_tail_bound(q, a, r)
            let n: Nat satisfy {
                metric_tail_bound(q, a, n, r)
            }
            lte_ref(n)
            n <= n
            metric_tail_bound_at(q, a, n, r, n)
            q(n).distance(a) < r
            open_ball_contains_iff(a, r, q(n))
            open_ball(a, r).contains(q(n)) = (q(n).distance(a) < r)
            open_ball(a, r).contains(q(n))
            subset_contains(open_ball(a, r), s.c, q(n))
            s.c.contains(q(n))
            compl_contains_eq(s, q(n))
            s.c.contains(q(n)) = not s.contains(q(n))
            not s.contains(q(n))
            s.contains(q(n))
            false
        }
        by_contradiction(s.contains(a))
        s.contains(a)
    }
}

/// The limit of a sequence of points of a closed interval lies in the
/// interval.
///
/// The classical argument: if the limit exceeded the upper endpoint, the tail
/// of the sequence would eventually lie strictly above it, contradicting the
/// interval membership of every term.
theorem tendsto_metric_in_closed_interval(q: Nat -> Real, l: Real, lo: Real, hi: Real) {
    (forall(n: Nat) { closed_interval_set(lo, hi).contains(q(n)) })
    and tendsto_metric(q, l)
    implies closed_interval_set(lo, hi).contains(l)
} by {
    if (forall(n: Nat) { closed_interval_set(lo, hi).contains(q(n)) })
        and tendsto_metric(q, l) {
        real_tendsto_metric_imp_converges_to(q, l)
        converges_to(q, l)
        if not l <= hi {
            not_lte_imp_gt(l, hi)
            hi < l
            lt_imp_minus_pos(hi, l)
            (l - hi).is_positive
            converges_to(q, l) = forall(eps: Real) {
                eps.is_positive implies exists(n: Nat) {
                    tail_bound(q, l, n, eps)
                }
            }
            exists(n: Nat) {
                tail_bound(q, l, n, l - hi)
            }
            let n: Nat satisfy {
                tail_bound(q, l, n, l - hi)
            }
            lte_ref(n)
            n <= n
            tail_bound_implies_is_close(q, l, n, l - hi, n)
            q(n).is_close(l, l - hi)
            close_imp_bounds(q(n), l, l - hi)
            q(n) > l - (l - hi)
            sub_minus_sub(l, hi)
            l - (l - hi) = hi
            q(n) > hi
            closed_interval_set_contains_eq(lo, hi, q(n))
            closed_interval_set(lo, hi).contains(q(n)) = closed_interval(lo, hi, q(n))
            closed_interval(lo, hi, q(n)) = (lo <= q(n) and q(n) <= hi)
            lo <= q(n) and q(n) <= hi
            q(n) <= hi
            lte_lt_trans(q(n), hi, q(n))
            q(n) < q(n)
            not_lt_self(q(n))
            false
        }
        by_contradiction(l <= hi)
        l <= hi
        if not lo <= l {
            not_lte_imp_gt(lo, l)
            l < lo
            lt_imp_minus_pos(l, lo)
            (lo - l).is_positive
            converges_to(q, l) = forall(eps: Real) {
                eps.is_positive implies exists(n: Nat) {
                    tail_bound(q, l, n, eps)
                }
            }
            exists(n: Nat) {
                tail_bound(q, l, n, lo - l)
            }
            let n: Nat satisfy {
                tail_bound(q, l, n, lo - l)
            }
            lte_ref(n)
            n <= n
            tail_bound_implies_is_close(q, l, n, lo - l, n)
            q(n).is_close(l, lo - l)
            close_imp_bounds(q(n), l, lo - l)
            q(n) < l + (lo - l)
            add_sub_cancel(l, lo)
            l + (lo - l) = lo
            q(n) < lo
            closed_interval_set_contains_eq(lo, hi, q(n))
            closed_interval_set(lo, hi).contains(q(n)) = closed_interval(lo, hi, q(n))
            closed_interval(lo, hi, q(n)) = (lo <= q(n) and q(n) <= hi)
            lo <= q(n) and q(n) <= hi
            lo <= q(n)
            lte_lt_trans(lo, q(n), lo)
            lo < lo
            not_lt_self(lo)
            false
        }
        by_contradiction(lo <= l)
        lo <= l
        closed_interval(lo, hi, l) = (lo <= l and l <= hi)
        closed_interval(lo, hi, l)
        closed_interval_set_contains_eq(lo, hi, l)
        closed_interval_set(lo, hi).contains(l) = closed_interval(lo, hi, l)
        closed_interval_set(lo, hi).contains(l)
    }
}

/// A closed subset of a complete metric space contains the limits of its
/// Cauchy sequences: closed subsets of complete spaces are complete.
///
/// Completeness supplies a limit in the ambient space, and the limit lies in
/// the closed set because closed sets are closed under limits of sequences.
theorem closed_set_in_complete_space_has_limit_in_set[M: CompleteMetricSpace](s: Set[M], q: Nat -> M) {
    is_closed_metric(s) and (forall(n: Nat) { s.contains(q(n)) }) and is_cauchy_metric(q)
    implies exists(a: M) { s.contains(a) and tendsto_metric(q, a) }
} by {
    if is_closed_metric(s) and (forall(n: Nat) { s.contains(q(n)) }) and is_cauchy_metric(q) {
        M.cauchy_metric_imp_tendsto(q)
        let a: M satisfy {
            tendsto_metric(q, a)
        }
        tendsto_metric_in_closed_set(s, q, a)
        s.contains(a)
        s.contains(a) and tendsto_metric(q, a)
        exists(b: M) { s.contains(b) and tendsto_metric(q, b) }
    }
}
