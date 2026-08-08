/// Closed metric balls are closed sets.

from analysis.metric.metric_space import MetricSpace, closed_ball, open_ball, is_open_metric,
    is_closed_metric, closed_ball_complement_has_open_neighborhood
from real import Real
from data.basic.logic import by_contradiction, not_forall_imp_exists_not

/// A closed ball is closed in the metric topology.
theorem closed_ball_is_closed_metric[M: MetricSpace](x: M, r: Real) {
    is_closed_metric(closed_ball(x, r))
} by {
    let s = closed_ball(x, r).c
    forall(y: M) {
        if s.contains(y) {
            closed_ball_complement_has_open_neighborhood(x, r, y)
            let rr: Real satisfy {
                rr.is_positive and open_ball(y, rr).subset(closed_ball(x, r).c)
            }
            exists(rr2: Real) {
                rr2.is_positive and open_ball(y, rr2).subset(s)
            }
        }
    }
    if not is_open_metric(s) {
        not_forall_imp_exists_not(function(y: M) {
            s.contains(y) implies exists(rr: Real) {
                rr.is_positive and open_ball(y, rr).subset(s)
            }
        })
        let w: M satisfy {
            not (s.contains(w) implies exists(rr: Real) {
                rr.is_positive and open_ball(w, rr).subset(s)
            })
        }
        exists(rr: Real) {
            rr.is_positive and open_ball(w, rr).subset(s)
        }
        false
    }
    by_contradiction(is_open_metric(s))
}
