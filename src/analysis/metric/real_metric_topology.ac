/// The metric topology on the real line is the standard topology.
///
/// The real line carries the `MetricSpace` structure of absolute distance and
/// the standard `TopologicalSpace` of interior-point open sets.  This file
/// proves the two notions of openness agree, so the real line is a
/// `MetricTopologicalSpace` and the generic metric-topology theorems apply to
/// it.  Closed intervals are then metric-closed, and closed intervals inherit
/// completeness from the real line.

from nat import Nat
from data.basic.set import Set, subset_contains
from data.basic.logic import by_contradiction, not_forall_imp_exists_not, not_implies
from real import Real
from analysis.metric.metric_space import MetricSpace, open_ball, open_ball_contains_iff,
    is_open_metric, is_closed_metric, is_cauchy_metric, tendsto_metric
from analysis.metric.metric_topology import MetricTopologicalSpace,
    metric_closed_eq_topological_closed
from analysis.metric.complete_metric_space import real_distance
from analysis.metric.metric_sequence import closed_set_in_complete_space_has_limit_in_set
from analysis.topology.borel_functions import real_is_open, real_is_interior_point,
    real_is_interior_point_intro
from analysis.topology.topological_space import is_closed
from analysis.topology.real_topology import closed_interval_set_is_closed
from order_set import closed_interval_set

numerals Real

/// A topologically open set of reals is metric-open: every point has an
/// interior-point ball, and interior-point balls are metric open balls.
theorem real_is_open_imp_is_open_metric(s: Set[Real]) {
    Real.is_open(s) implies is_open_metric(s)
} by {
    if Real.is_open(s) {
        is_open_metric(s) = forall(x: Real) {
            s.contains(x) implies exists(r: Real) {
                r.is_positive and open_ball(x, r).subset(s)
            }
        }
        forall(x: Real) {
            if s.contains(x) {
                Real.is_open(s) = real_is_open(s)
                real_is_open(s) = forall(y: Real) {
                    s.contains(y) implies real_is_interior_point(s, y)
                }
                real_is_interior_point(s, x)
                real_is_interior_point(s, x) = (s.contains(x) and exists(eps: Real) {
                    eps.is_positive and forall(y: Real) {
                        y.is_close(x, eps) implies s.contains(y)
                    }
                })
                exists(eps: Real) {
                    eps.is_positive and forall(y: Real) {
                        y.is_close(x, eps) implies s.contains(y)
                    }
                }
                let eps: Real satisfy {
                    eps.is_positive and forall(y: Real) {
                        y.is_close(x, eps) implies s.contains(y)
                    }
                }
                forall(y: Real) {
                    if open_ball(x, eps).contains(y) {
                        open_ball_contains_iff(x, eps, y)
                        open_ball(x, eps).contains(y) = (y.distance(x) < eps)
                        y.distance(x) < eps
                        y.distance(x) = real_distance(y, x)
                        real_distance(y, x) = (y - x).abs
                        (y - x).abs < eps
                        y.is_close(x, eps)
                        s.contains(y)
                    }
                }
                open_ball(x, eps).subset(s)
                exists(r: Real) {
                    r.is_positive and open_ball(x, r).subset(s)
                }
            }
        }
        if not is_open_metric(s) {
            is_open_metric(s) = forall(x: Real) {
                s.contains(x) implies exists(r: Real) {
                    r.is_positive and open_ball(x, r).subset(s)
                }
            }
            not forall(x: Real) {
                s.contains(x) implies exists(r: Real) {
                    r.is_positive and open_ball(x, r).subset(s)
                }
            }
            not_forall_imp_exists_not[Real](function(x: Real) {
                s.contains(x) implies exists(r: Real) {
                    r.is_positive and open_ball(x, r).subset(s)
                }
            })
            let x: Real satisfy {
                not (s.contains(x) implies exists(r: Real) {
                    r.is_positive and open_ball(x, r).subset(s)
                })
            }
            not_implies(s.contains(x), exists(r: Real) {
                r.is_positive and open_ball(x, r).subset(s)
            })
            (not (s.contains(x) implies exists(r: Real) {
                r.is_positive and open_ball(x, r).subset(s)
            })) = (s.contains(x) and not exists(r: Real) {
                r.is_positive and open_ball(x, r).subset(s)
            })
            s.contains(x) and not exists(r: Real) {
                r.is_positive and open_ball(x, r).subset(s)
            }
            s.contains(x)
            exists(r: Real) {
                r.is_positive and open_ball(x, r).subset(s)
            }
            not exists(r: Real) {
                r.is_positive and open_ball(x, r).subset(s)
            }
            false
        }
        by_contradiction(is_open_metric(s))
        is_open_metric(s)
    }
}

/// A metric-open set of reals is topologically open: metric open balls are
/// interior-point balls.
theorem is_open_metric_imp_real_is_open(s: Set[Real]) {
    is_open_metric(s) implies Real.is_open(s)
} by {
    if is_open_metric(s) {
        Real.is_open(s) = real_is_open(s)
        real_is_open(s) = forall(x: Real) {
            s.contains(x) implies real_is_interior_point(s, x)
        }
        forall(x: Real) {
            if s.contains(x) {
                is_open_metric(s) = forall(y: Real) {
                    s.contains(y) implies exists(r: Real) {
                        r.is_positive and open_ball(y, r).subset(s)
                    }
                }
                exists(r: Real) {
                    r.is_positive and open_ball(x, r).subset(s)
                }
                let r: Real satisfy {
                    r.is_positive and open_ball(x, r).subset(s)
                }
                forall(y: Real) {
                    if y.is_close(x, r) {
                        y.is_close(x, r) = ((y - x).abs < r)
                        (y - x).abs < r
                        real_distance(y, x) = (y - x).abs
                        real_distance(y, x) < r
                        y.distance(x) = real_distance(y, x)
                        y.distance(x) < r
                        open_ball_contains_iff(x, r, y)
                        open_ball(x, r).contains(y) = (y.distance(x) < r)
                        open_ball(x, r).contains(y)
                        subset_contains(open_ball(x, r), s, y)
                        s.contains(y)
                    }
                }
                real_is_interior_point_intro(s, x, r)
                real_is_interior_point(s, x)
            }
        }
        real_is_open(s)
        Real.is_open(s)
    }
}

/// Topological openness on the real line is metric openness.
theorem real_is_open_eq_is_open_metric(s: Set[Real]) {
    Real.is_open(s) = is_open_metric(s)
} by {
    if Real.is_open(s) {
        real_is_open_imp_is_open_metric(s)
        is_open_metric(s)
    }
    if is_open_metric(s) {
        is_open_metric_imp_real_is_open(s)
        Real.is_open(s)
    }
}

/// The real line is a metric topological space: the metric and topological
/// open sets agree.
instance Real: MetricTopologicalSpace

/// Closed intervals of the real line are metric-closed.
theorem closed_interval_is_closed_metric(lo: Real, hi: Real) {
    is_closed_metric(closed_interval_set(lo, hi))
} by {
    closed_interval_set_is_closed(lo, hi)
    is_closed[Real](closed_interval_set(lo, hi))
    metric_closed_eq_topological_closed[Real](closed_interval_set(lo, hi))
    is_closed_metric(closed_interval_set(lo, hi))
}

/// A Cauchy sequence in a closed interval of the real line converges to a
/// point of the interval: closed intervals are complete.
theorem closed_interval_cauchy_metric_has_limit_in_interval(q: Nat -> Real, lo: Real, hi: Real) {
    (forall(n: Nat) { closed_interval_set(lo, hi).contains(q(n)) }) and is_cauchy_metric(q)
    implies exists(a: Real) { closed_interval_set(lo, hi).contains(a) and tendsto_metric(q, a) }
} by {
    if (forall(n: Nat) { closed_interval_set(lo, hi).contains(q(n)) }) and is_cauchy_metric(q) {
        closed_interval_is_closed_metric(lo, hi)
        closed_set_in_complete_space_has_limit_in_set[Real](closed_interval_set(lo, hi), q)
        exists(a: Real) { closed_interval_set(lo, hi).contains(a) and tendsto_metric(q, a) }
    }
}
