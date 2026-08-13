/// Separation and compactness in metric spaces.
///
/// The metric topology is Hausdorff: two distinct points have disjoint open
/// balls around them.  Consequently, in a metric space compatible with the
/// generic topology framework, compact sets are closed.  A compact set is also
/// bounded: the unit balls around its points form an open cover, and a finite
/// subcover exhibits it inside a finite union of bounded balls.

from list import List
from data.list.list_cons_membership import cons_contains_head,
    cons_contains_of_tail_contains, nil_not_contains
from data.basic.set import Set, subset_contains, subset_contains_eq, union_contains_eq
from analysis.metric.metric_space import MetricSpace, is_open_metric, is_closed_metric,
    open_ball, open_ball_contains_iff, open_ball_is_open_metric, center_in_open_ball,
    distance_non_negative, distance_zero_iff_eq, is_bounded, open_ball_is_bounded,
    union_of_bounded_is_bounded, subset_of_bounded_is_bounded, empty_set_is_bounded
from analysis.metric.metric_topology import MetricTopologicalSpace,
    metric_open_is_topological_open, metric_closed_eq_topological_closed,
    open_ball_is_topological_open
from analysis.topology.topological_space import TopologicalSpace, big_union,
    big_union_contains_of_member, is_open_cover, is_open_cover_iff, is_compact,
    is_subfamily, is_subfamily_cons_mp, list_union_of_sets,
    list_union_of_sets_nil, list_union_of_sets_cons, is_hausdorff, is_hausdorff_iff,
    is_closed
from analysis.topology.compact_hausdorff import compact_imp_closed_in_hausdorff
from analysis.topology.compact_union import compact_open_cover_has_finite_subcover
from real import Real, eps_lt_half
from order import lt_trans, lte_lt_trans, lte_both_ways_imp_eq, not_lt_imp_gte, not_lt_self
from data.basic.logic import by_contradiction, not_forall_imp_exists_not, not_implies
numerals Real

/// Distinct points of a metric space have disjoint open balls around them.
theorem metric_hausdorff_separating_balls[M: MetricSpace](x: M, y: M) {
    x != y implies exists(u: Set[M], v: Set[M]) {
        is_open_metric(u) and is_open_metric(v) and u.contains(x) and v.contains(y)
            and u.is_disjoint(v)
    }
} by {
    if x != y {
        distance_zero_iff_eq(x, y)
        (x.distance(y) = 0) = (x = y)
        not (x.distance(y) = 0)
        distance_non_negative(x, y)
        not x.distance(y).is_negative
        Real.0 <= x.distance(y)
        if not (Real.0 < x.distance(y)) {
            not_lt_imp_gte(Real.0, x.distance(y))
            Real.0 >= x.distance(y)
            x.distance(y) <= Real.0
            lte_both_ways_imp_eq(Real.0, x.distance(y))
            Real.0 = x.distance(y)
            x.distance(y) = Real.0
            not (x.distance(y) = Real.0)
            false
        }
        Real.0 < x.distance(y)
        eps_lt_half(x.distance(y))
        let half: Real satisfy {
            half.is_positive and half + half < x.distance(y)
        }
        let u: Set[M] = open_ball(x, half)
        let v: Set[M] = open_ball(y, half)
        open_ball_is_open_metric(x, half)
        is_open_metric(u)
        open_ball_is_open_metric(y, half)
        is_open_metric(v)
        center_in_open_ball(x, half)
        half.is_positive implies open_ball(x, half).contains(x)
        u.contains(x)
        center_in_open_ball(y, half)
        half.is_positive implies open_ball(y, half).contains(y)
        v.contains(y)
        u.is_disjoint(v) = forall(z: M) {
            not (u.contains(z) and v.contains(z))
        }
        forall(z: M) {
            if u.contains(z) and v.contains(z) {
                open_ball_contains_iff(x, half, z)
                u.contains(z) = (z.distance(x) < half)
                z.distance(x) < half
                open_ball_contains_iff(y, half, z)
                v.contains(z) = (z.distance(y) < half)
                z.distance(y) < half
                x.distance(y) <= x.distance(z) + z.distance(y)
                x.distance(z) = z.distance(x)
                x.distance(z) + z.distance(y) < half + half
                lte_lt_trans(x.distance(y), x.distance(z) + z.distance(y), half + half)
                x.distance(y) < half + half
                lt_trans(x.distance(y), half + half, x.distance(y))
                x.distance(y) < x.distance(y)
                not_lt_self(x.distance(y))
                false
            }
            not (u.contains(z) and v.contains(z))
        }
        u.is_disjoint(v)
        (is_open_metric(u) and is_open_metric(v) and u.contains(x) and v.contains(y)
            and u.is_disjoint(v))
        exists(uu: Set[M], vv: Set[M]) {
            is_open_metric(uu) and is_open_metric(vv) and uu.contains(x) and vv.contains(y)
                and uu.is_disjoint(vv)
        }
    }
}

/// The metric topology is Hausdorff: distinct points have disjoint open balls.
theorem metric_topology_hausdorff[M: MetricSpace] {
    is_hausdorff[M](is_open_metric)
} by {
    if not is_hausdorff[M](is_open_metric) {
        is_hausdorff_iff[M](is_open_metric)
        is_hausdorff[M](is_open_metric) = forall(x: M, y: M) {
            x != y implies exists(u: Set[M], v: Set[M]) {
                is_open_metric(u) and is_open_metric(v) and u.contains(x) and v.contains(y)
                    and u.is_disjoint(v)
            }
        }
        not forall(x: M, y: M) {
            x != y implies exists(u: Set[M], v: Set[M]) {
                is_open_metric(u) and is_open_metric(v) and u.contains(x) and v.contains(y)
                    and u.is_disjoint(v)
            }
        }
        not_forall_imp_exists_not[M](function(x: M) {
            forall(y: M) {
                x != y implies exists(u: Set[M], v: Set[M]) {
                    is_open_metric(u) and is_open_metric(v) and u.contains(x) and v.contains(y)
                        and u.is_disjoint(v)
                }
            }
        })
        let x: M satisfy {
            not forall(y: M) {
                x != y implies exists(u: Set[M], v: Set[M]) {
                    is_open_metric(u) and is_open_metric(v) and u.contains(x) and v.contains(y)
                        and u.is_disjoint(v)
                }
            }
        }
        not_forall_imp_exists_not[M](function(y: M) {
            x != y implies exists(u: Set[M], v: Set[M]) {
                is_open_metric(u) and is_open_metric(v) and u.contains(x) and v.contains(y)
                    and u.is_disjoint(v)
            }
        })
        let y: M satisfy {
            not (x != y implies exists(u: Set[M], v: Set[M]) {
                is_open_metric(u) and is_open_metric(v) and u.contains(x) and v.contains(y)
                    and u.is_disjoint(v)
            })
        }
        not_implies(x != y, exists(u: Set[M], v: Set[M]) {
            is_open_metric(u) and is_open_metric(v) and u.contains(x) and v.contains(y)
                and u.is_disjoint(v)
        })
        (not (x != y implies exists(u: Set[M], v: Set[M]) {
            is_open_metric(u) and is_open_metric(v) and u.contains(x) and v.contains(y)
                and u.is_disjoint(v)
        })) = (x != y and not exists(u: Set[M], v: Set[M]) {
            is_open_metric(u) and is_open_metric(v) and u.contains(x) and v.contains(y)
                and u.is_disjoint(v)
        })
        x != y and not exists(u: Set[M], v: Set[M]) {
            is_open_metric(u) and is_open_metric(v) and u.contains(x) and v.contains(y)
                and u.is_disjoint(v)
        }
        x != y
        metric_hausdorff_separating_balls(x, y)
        exists(u: Set[M], v: Set[M]) {
            is_open_metric(u) and is_open_metric(v) and u.contains(x) and v.contains(y)
                and u.is_disjoint(v)
        }
        not exists(u: Set[M], v: Set[M]) {
            is_open_metric(u) and is_open_metric(v) and u.contains(x) and v.contains(y)
                and u.is_disjoint(v)
        }
        false
    }
    by_contradiction(is_hausdorff[M](is_open_metric))
    (not is_hausdorff[M](is_open_metric) implies false) implies is_hausdorff[M](is_open_metric)
    is_hausdorff[M](is_open_metric)
}

/// Every metric space type is inhabited: some point exists.
theorem metric_space_point_exists[M: MetricSpace] {
    exists(x: M) {
        true
    }
} by {
}

/// Distinct points of a compatible metric topological space have disjoint open
/// neighborhoods.
theorem metric_topological_separating_opens[M: MetricTopologicalSpace](x: M, y: M) {
    x != y implies exists(u: Set[M], v: Set[M]) {
        M.is_open(u) and M.is_open(v) and u.contains(x) and v.contains(y)
            and u.is_disjoint(v)
    }
} by {
    if x != y {
        metric_hausdorff_separating_balls(x, y)
        exists(u: Set[M], v: Set[M]) {
            is_open_metric(u) and is_open_metric(v) and u.contains(x) and v.contains(y)
                and u.is_disjoint(v)
        }
        let u: Set[M] satisfy {
            exists(v: Set[M]) {
                is_open_metric(u) and is_open_metric(v) and u.contains(x) and v.contains(y)
                    and u.is_disjoint(v)
            }
        }
        let v: Set[M] satisfy {
            is_open_metric(u) and is_open_metric(v) and u.contains(x) and v.contains(y)
                and u.is_disjoint(v)
        }
        metric_open_is_topological_open(u)
        is_open_metric(u) implies M.is_open(u)
        M.is_open(u)
        metric_open_is_topological_open(v)
        is_open_metric(v) implies M.is_open(v)
        M.is_open(v)
        (M.is_open(u) and M.is_open(v) and u.contains(x) and v.contains(y)
            and u.is_disjoint(v))
        exists(uu: Set[M], vv: Set[M]) {
            M.is_open(uu) and M.is_open(vv) and uu.contains(x) and vv.contains(y)
                and uu.is_disjoint(vv)
        }
    }
}

/// The topological openness of a compatible metric topological space is Hausdorff.
theorem metric_topological_hausdorff[M: MetricTopologicalSpace] {
    is_hausdorff[M](M.is_open)
} by {
    if not is_hausdorff[M](M.is_open) {
        is_hausdorff_iff[M](M.is_open)
        is_hausdorff[M](M.is_open) = forall(x: M, y: M) {
            x != y implies exists(u: Set[M], v: Set[M]) {
                M.is_open(u) and M.is_open(v) and u.contains(x) and v.contains(y)
                    and u.is_disjoint(v)
            }
        }
        not forall(x: M, y: M) {
            x != y implies exists(u: Set[M], v: Set[M]) {
                M.is_open(u) and M.is_open(v) and u.contains(x) and v.contains(y)
                    and u.is_disjoint(v)
            }
        }
        not_forall_imp_exists_not[M](function(x: M) {
            forall(y: M) {
                x != y implies exists(u: Set[M], v: Set[M]) {
                    M.is_open(u) and M.is_open(v) and u.contains(x) and v.contains(y)
                        and u.is_disjoint(v)
                }
            }
        })
        let x: M satisfy {
            not forall(y: M) {
                x != y implies exists(u: Set[M], v: Set[M]) {
                    M.is_open(u) and M.is_open(v) and u.contains(x) and v.contains(y)
                        and u.is_disjoint(v)
                }
            }
        }
        not_forall_imp_exists_not[M](function(y: M) {
            x != y implies exists(u: Set[M], v: Set[M]) {
                M.is_open(u) and M.is_open(v) and u.contains(x) and v.contains(y)
                    and u.is_disjoint(v)
            }
        })
        let y: M satisfy {
            not (x != y implies exists(u: Set[M], v: Set[M]) {
                M.is_open(u) and M.is_open(v) and u.contains(x) and v.contains(y)
                    and u.is_disjoint(v)
            })
        }
        not_implies(x != y, exists(u: Set[M], v: Set[M]) {
            M.is_open(u) and M.is_open(v) and u.contains(x) and v.contains(y)
                and u.is_disjoint(v)
        })
        (not (x != y implies exists(u: Set[M], v: Set[M]) {
            M.is_open(u) and M.is_open(v) and u.contains(x) and v.contains(y)
                and u.is_disjoint(v)
        })) = (x != y and not exists(u: Set[M], v: Set[M]) {
            M.is_open(u) and M.is_open(v) and u.contains(x) and v.contains(y)
                and u.is_disjoint(v)
        })
        x != y and not exists(u: Set[M], v: Set[M]) {
            M.is_open(u) and M.is_open(v) and u.contains(x) and v.contains(y)
                and u.is_disjoint(v)
        }
        x != y
        metric_topological_separating_opens(x, y)
        exists(u: Set[M], v: Set[M]) {
            M.is_open(u) and M.is_open(v) and u.contains(x) and v.contains(y)
                and u.is_disjoint(v)
        }
        not exists(u: Set[M], v: Set[M]) {
            M.is_open(u) and M.is_open(v) and u.contains(x) and v.contains(y)
                and u.is_disjoint(v)
        }
        false
    }
    by_contradiction(is_hausdorff[M](M.is_open))
    (not is_hausdorff[M](M.is_open) implies false) implies is_hausdorff[M](M.is_open)
    is_hausdorff[M](M.is_open)
}

/// A compact subset of a metric space is closed.
theorem compact_imp_closed_metric[M: MetricTopologicalSpace](s: Set[M]) {
    is_compact[M](s) implies is_closed_metric(s)
} by {
    if is_compact[M](s) {
        metric_topological_hausdorff[M]
        compact_imp_closed_in_hausdorff[M](s)
        is_closed[M](s)
        metric_closed_eq_topological_closed(s)
        is_closed_metric(s)
    }
}

/// True if a set is the unit ball around a point of `s`.
define unit_ball_cover_family[M: MetricSpace](s: Set[M], v: Set[M]) -> Bool {
    exists(y: M) {
        s.contains(y) and v = open_ball(y, Real.1)
    }
}

/// Unfolding lemma for `unit_ball_cover_family`.
theorem unit_ball_cover_family_iff[M: MetricSpace](s: Set[M], v: Set[M]) {
    unit_ball_cover_family(s, v) = exists(y: M) {
        s.contains(y) and v = open_ball(y, Real.1)
    }
} by {
}

/// The union of the sets in a finite list of bounded sets is bounded.
theorem bounded_union_list[M: MetricSpace](items: List[Set[M]]) {
    (forall(k: Set[M]) { items.contains(k) implies is_bounded(k) })
        implies is_bounded(list_union_of_sets[M](items))
} by {
    define p(xs: List[Set[M]]) -> Bool {
        (forall(k: Set[M]) { xs.contains(k) implies is_bounded(k) })
            implies is_bounded(list_union_of_sets[M](xs))
    }
    if forall(k: Set[M]) { List.nil[Set[M]].contains(k) implies is_bounded(k) } {
        list_union_of_sets_nil[M]
        metric_space_point_exists[M]
        let x0: M satisfy {
            true
        }
        empty_set_is_bounded[M](x0)
        is_bounded(list_union_of_sets[M](List.nil[Set[M]]))
    }
    p(List.nil[Set[M]])
    forall(head: Set[M], tail: List[Set[M]]) {
        if p(tail) {
            if forall(k: Set[M]) { List.cons(head, tail).contains(k) implies is_bounded(k) } {
                cons_contains_head[Set[M]](head, tail)
                forall(k: Set[M]) { List.cons(head, tail).contains(k) implies is_bounded(k) }
                is_bounded(head)
                forall(k: Set[M]) {
                    if tail.contains(k) {
                        cons_contains_of_tail_contains[Set[M]](head, tail, k)
                        forall(w: Set[M]) { List.cons(head, tail).contains(w) implies is_bounded(w) }
                        is_bounded(k)
                    }
                }
                p(tail) = ((forall(k: Set[M]) { tail.contains(k) implies is_bounded(k) })
                    implies is_bounded(list_union_of_sets[M](tail)))
                forall(k: Set[M]) { tail.contains(k) implies is_bounded(k) }
                is_bounded(list_union_of_sets[M](tail))
                union_of_bounded_is_bounded(head, list_union_of_sets[M](tail))
                is_bounded(head.union(list_union_of_sets[M](tail)))
                list_union_of_sets_cons[M](head, tail)
                is_bounded(list_union_of_sets[M](List.cons(head, tail)))
            }
            p(List.cons(head, tail))
        }
        p(tail) implies p(List.cons(head, tail))
    }
    p(List.nil[Set[M]]) and forall(head: Set[M], tail: List[Set[M]]) {
        p(tail) implies p(List.cons(head, tail))
    }
    List.induction(function(xs: List[Set[M]]) { p(xs) })
    forall(xs: List[Set[M]]) { p(xs) }
    p(items) = ((forall(k: Set[M]) { items.contains(k) implies is_bounded(k) })
        implies is_bounded(list_union_of_sets[M](items)))
    if forall(k: Set[M]) { items.contains(k) implies is_bounded(k) } {
        is_bounded(list_union_of_sets[M](items))
    }
}

/// A member of a finite subfamily of the unit-ball cover belongs to the cover.
theorem subfamily_member_imp_family_unit_ball[M: MetricSpace](
    s: Set[M], items: List[Set[M]], k: Set[M]) {
    is_subfamily[M](unit_ball_cover_family(s), items) and items.contains(k)
        implies unit_ball_cover_family(s, k)
} by {
    define p(xs: List[Set[M]]) -> Bool {
        is_subfamily[M](unit_ball_cover_family(s), xs) and xs.contains(k)
            implies unit_ball_cover_family(s, k)
    }
    if is_subfamily[M](unit_ball_cover_family(s), List.nil[Set[M]]) and List.nil[Set[M]].contains(k) {
        nil_not_contains[Set[M]](k)
        false
    }
    p(List.nil[Set[M]])
    forall(head: Set[M], tail: List[Set[M]]) {
        if p(tail) {
            if is_subfamily[M](unit_ball_cover_family(s), List.cons(head, tail))
                and List.cons(head, tail).contains(k) {
                is_subfamily_cons_mp[M](unit_ball_cover_family(s), head, tail)
                if head = k {
                    unit_ball_cover_family(s, k)
                } else {
                    unit_ball_cover_family(s, k)
                }
            }
            p(List.cons(head, tail))
        }
        p(tail) implies p(List.cons(head, tail))
    }
    p(List.nil[Set[M]]) and forall(head: Set[M], tail: List[Set[M]]) {
        p(tail) implies p(List.cons(head, tail))
    }
    List.induction(function(xs: List[Set[M]]) { p(xs) })
    forall(xs: List[Set[M]]) { p(xs) }
    if is_subfamily[M](unit_ball_cover_family(s), items) and items.contains(k) {
        unit_ball_cover_family(s, k)
    }
}

/// A member of a finite subfamily of the unit-ball cover is a bounded unit ball.
theorem subfamily_member_is_unit_ball[M: MetricTopologicalSpace](
    s: Set[M], items: List[Set[M]], k: Set[M]) {
    is_subfamily[M](unit_ball_cover_family(s), items) and items.contains(k) implies is_bounded(k)
} by {
    if is_subfamily[M](unit_ball_cover_family(s), items) and items.contains(k) {
        subfamily_member_imp_family_unit_ball(s, items, k)
        unit_ball_cover_family(s, k)
        unit_ball_cover_family_iff(s, k)
        let y: M satisfy {
            s.contains(y) and k = open_ball(y, Real.1)
        }
        open_ball_is_bounded(y, Real.1)
        is_bounded(open_ball(y, Real.1))
        is_bounded(k)
    }
}

/// A compact subset of a metric space is bounded: unit balls around its points
/// cover it, and a finite subcover lies inside a finite union of bounded sets.
theorem compact_imp_bounded_metric[M: MetricTopologicalSpace](s: Set[M]) {
    is_compact[M](s) implies is_bounded(s)
} by {
    if is_compact[M](s) {
        let c: Set[M] -> Bool = unit_ball_cover_family(s)
        forall(v: Set[M]) {
            if c(v) {
                unit_ball_cover_family_iff(s, v)
                let y: M satisfy {
                    s.contains(y) and v = open_ball(y, Real.1)
                }
                open_ball_is_topological_open[M](y, Real.1)
                M.is_open(open_ball(y, Real.1))
                M.is_open(v)
            }
        }
        forall(y: M) {
            if s.contains(y) {
                unit_ball_cover_family_iff(s, open_ball(y, Real.1))
                c(open_ball(y, Real.1))
                Real.1.is_positive
                center_in_open_ball(y, Real.1)
                open_ball(y, Real.1).contains(y)
                big_union_contains_of_member(c, open_ball(y, Real.1), y)
                big_union(c).contains(y)
            }
        }
        subset_contains_eq[M](s, big_union(c))
        s.subset(big_union(c))
        is_open_cover_iff[M](c, s)
        is_open_cover[M](c, s)
        compact_open_cover_has_finite_subcover[M](c, s)
        let items: List[Set[M]] satisfy {
            is_subfamily[M](c, items) and s.subset(list_union_of_sets[M](items))
        }
        forall(k: Set[M]) {
            if items.contains(k) {
                subfamily_member_is_unit_ball(s, items, k)
                is_bounded(k)
            }
        }
        bounded_union_list(items)
        is_bounded(list_union_of_sets[M](items))
        subset_of_bounded_is_bounded(s, list_union_of_sets[M](items))
        is_bounded(s)
    }
}
