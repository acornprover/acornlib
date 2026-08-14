/// The Banach fixed point theorem for contractions of complete metric spaces.
///
/// A contraction of a complete metric space has a unique fixed point, and the
/// orbit of every starting point converges to it.  The proof follows the
/// classical route: the consecutive distances along an orbit are dominated by
/// the geometric sequence `c^n · d(f(x0), x0)`, so the orbit is Cauchy by the
/// convergence of the geometric series, completeness supplies a limit, and the
/// contraction inequality forces the limit to be a fixed point.

from nat import Nat, add_sub, pow_zero, pow_one, pow_add
from list import partial, partial_zero, partial_split_last
from real import Real, converges, mul_seq, sub_zero_imp_eq, is_cauchy_seq,
    is_cauchy_seq_at, cauchy_bound, mul_one_left, lte_abs, abs_of_nonneg, geom_converges,
    converges_mul_seq, gt_zero_imp_pos, lt_imp_minus_pos
from order import not_lt_self, lte_lt_trans, lt_trans, lte_trans, not_lt_imp_gte,
    eq_or_lt_of_lte, not_lte_imp_gt, lt_imp_lte
from ordered_field import mul_le_mul_of_nonneg_left, mul_lt_mul_of_pos_left
from algebra.add_ordered_group import add_le_add_right, add_lt_add_left, add_le_add
from data.basic.functions import function_extensionality
from analysis.real.real_geometric_comparison import partial_mul_seq
from analysis.metric.metric_space import MetricSpace, distance_non_negative,
    cauchy_metric_bound, cauchy_metric_bound_from_distances, is_cauchy_metric,
    is_cauchy_metric_from_bounds, metric_tail_bound, metric_tail_bound_at,
    tendsto_metric, tendsto_metric_tail_bound
from analysis.metric.complete_metric_space import CompleteMetricSpace, real_distance
from analysis.dynamical.dynamical_systems import iterate, iterate_zero, iterate_suc,
    iterate_suc_inner, is_fixed_point

numerals Real
numerals Nat

/// True if `f` is a contraction of the metric space `M` with constant `c`:
/// the constant lies in `[0, 1)` and `f` shrinks distances by the factor `c`.
define is_contraction[M: MetricSpace](f: M -> M, c: Real) -> Bool {
    Real.0 <= c and c < Real.1 and forall(x: M, y: M) {
        f(x).distance(f(y)) <= c * x.distance(y)
    }
}

/// The constant of a contraction is nonnegative.
theorem contraction_constant_nonneg[M: MetricSpace](f: M -> M, c: Real) {
    is_contraction(f, c) implies Real.0 <= c
} by {
    if is_contraction(f, c) {
        is_contraction(f, c) = (Real.0 <= c and c < Real.1 and forall(x: M, y: M) {
            f(x).distance(f(y)) <= c * x.distance(y)
        })
        Real.0 <= c
    }
}

/// The constant of a contraction is strictly below one.
theorem contraction_constant_lt_one[M: MetricSpace](f: M -> M, c: Real) {
    is_contraction(f, c) implies c < Real.1
} by {
    if is_contraction(f, c) {
        is_contraction(f, c) = (Real.0 <= c and c < Real.1 and forall(x: M, y: M) {
            f(x).distance(f(y)) <= c * x.distance(y)
        })
        c < Real.1
    }
}

/// A contraction shrinks every distance by the factor `c`.
theorem contraction_distance[M: MetricSpace](f: M -> M, c: Real, x: M, y: M) {
    is_contraction(f, c) implies f(x).distance(f(y)) <= c * x.distance(y)
} by {
    if is_contraction(f, c) {
        is_contraction(f, c) = (Real.0 <= c and c < Real.1 and forall(a: M, b: M) {
            f(a).distance(f(b)) <= c * a.distance(b)
        })
        forall(a: M, b: M) {
            f(a).distance(f(b)) <= c * a.distance(b)
        }
        f(x).distance(f(y)) <= c * x.distance(y)
    }
}

/// The orbit of `f` from `x0`: the sequence `x0, f(x0), f(f(x0)), ...`.
define orbit_seq[M: MetricSpace](f: M -> M, x0: M, n: Nat) -> M {
    iterate(f, n, x0)
}

/// The zeroth orbit point is the starting point.
theorem orbit_seq_zero[M: MetricSpace](f: M -> M, x0: M) {
    orbit_seq(f, x0, Nat.0) = x0
} by {
    iterate_zero(f, x0)
    iterate(f, Nat.0, x0) = x0
    orbit_seq(f, x0, Nat.0) = iterate(f, Nat.0, x0)
    orbit_seq(f, x0, Nat.0) = x0
}

/// Each orbit point is the image of the previous one.
theorem orbit_seq_suc[M: MetricSpace](f: M -> M, x0: M, n: Nat) {
    orbit_seq(f, x0, n.suc) = f(orbit_seq(f, x0, n))
} by {
    iterate_suc(f, n, x0)
    iterate(f, n.suc, x0) = f(iterate(f, n, x0))
    orbit_seq(f, x0, n.suc) = iterate(f, n.suc, x0)
    orbit_seq(f, x0, n) = iterate(f, n, x0)
    orbit_seq(f, x0, n.suc) = f(orbit_seq(f, x0, n))
}

/// The `n`-th partial sum of the geometric majorant of the consecutive
/// distances: `Σ_{k=0}^{n-1} c^k · d(f(x0), x0)`.
define orbit_partial[M: MetricSpace](f: M -> M, c: Real, x0: M, n: Nat) -> Real {
    partial(mul_seq(f(x0).distance(x0), c.pow), n)
}

/// Iterating a contraction `n` times shrinks distances by `c^n`.
theorem contraction_iterate_distance[M: MetricSpace](f: M -> M, c: Real, n: Nat, x: M, y: M) {
    is_contraction(f, c)
    implies iterate(f, n, x).distance(iterate(f, n, y)) <= c.pow(n) * x.distance(y)
} by {
    if is_contraction(f, c) {
        define p(k: Nat) -> Bool {
            iterate(f, k, x).distance(iterate(f, k, y)) <= c.pow(k) * x.distance(y)
        }
        iterate(f, Nat.0, x) = x
        iterate(f, Nat.0, y) = y
        iterate(f, Nat.0, x).distance(iterate(f, Nat.0, y)) = x.distance(y)
        c.pow(Nat.0) = Real.1
        Real.1 * x.distance(y) = x.distance(y)
        mul_one_left(x.distance(y))
        c.pow(Nat.0) * x.distance(y) = x.distance(y)
        iterate(f, Nat.0, x).distance(iterate(f, Nat.0, y)) <= c.pow(Nat.0) * x.distance(y)
        p(Nat.0)
        forall(k: Nat) {
            if p(k) {
                iterate(f, k.suc, x) = f(iterate(f, k, x))
                iterate(f, k.suc, y) = f(iterate(f, k, y))
                contraction_distance(f, c, iterate(f, k, x), iterate(f, k, y))
                f(iterate(f, k, x)).distance(f(iterate(f, k, y))) <= c * iterate(f, k, x).distance(iterate(f, k, y))
                iterate(f, k.suc, x).distance(iterate(f, k.suc, y)) <= c * iterate(f, k, x).distance(iterate(f, k, y))
                contraction_constant_nonneg(f, c)
                Real.0 <= c
                iterate(f, k, x).distance(iterate(f, k, y)) <= c.pow(k) * x.distance(y)
                mul_le_mul_of_nonneg_left(iterate(f, k, x).distance(iterate(f, k, y)), c.pow(k) * x.distance(y), c)
                c * iterate(f, k, x).distance(iterate(f, k, y)) <= c * (c.pow(k) * x.distance(y))
                lte_trans(iterate(f, k.suc, x).distance(iterate(f, k.suc, y)), c * iterate(f, k, x).distance(iterate(f, k, y)), c * (c.pow(k) * x.distance(y)))
                iterate(f, k.suc, x).distance(iterate(f, k.suc, y)) <= c * (c.pow(k) * x.distance(y))
                c * (c.pow(k) * x.distance(y)) = (c * c.pow(k)) * x.distance(y)
                pow_add(c, k, Nat.1)
                c.pow(k) * c.pow(Nat.1) = c.pow(k + Nat.1)
                pow_one(c)
                c.pow(Nat.1) = c
                c.pow(k) * c = c.pow(k + Nat.1)
                c * c.pow(k) = c.pow(k + Nat.1)
                c.pow(k + Nat.1) = c.pow(k.suc)
                c * c.pow(k) = c.pow(k.suc)
                (c * c.pow(k)) * x.distance(y) = c.pow(k.suc) * x.distance(y)
                c * (c.pow(k) * x.distance(y)) = c.pow(k.suc) * x.distance(y)
                lte_trans(iterate(f, k.suc, x).distance(iterate(f, k.suc, y)), c * (c.pow(k) * x.distance(y)), c.pow(k.suc) * x.distance(y))
                iterate(f, k.suc, x).distance(iterate(f, k.suc, y)) <= c.pow(k.suc) * x.distance(y)
                p(k.suc)
            }
        }
        p(Nat.0) and forall(k: Nat) {
            p(k) implies p(k.suc)
        }
        Nat.induction(p)
        p(n)
        iterate(f, n, x).distance(iterate(f, n, y)) <= c.pow(n) * x.distance(y)
    }
}

/// Consecutive orbit distances are dominated by the geometric majorant.
///
/// The distance from the `n`-th iterate to the next is at most
/// `c^n · d(f(x0), x0)`.
theorem orbit_consecutive_bound[M: MetricSpace](f: M -> M, c: Real, x0: M, n: Nat) {
    is_contraction(f, c)
    implies orbit_seq(f, x0, n.suc).distance(orbit_seq(f, x0, n)) <=
        mul_seq(f(x0).distance(x0), c.pow, n)
} by {
    if is_contraction(f, c) {
        iterate_suc_inner(f, n, x0)
        iterate(f, n.suc, x0) = iterate(f, n, f(x0))
        orbit_seq(f, x0, n.suc) = iterate(f, n.suc, x0)
        orbit_seq(f, x0, n.suc) = iterate(f, n, f(x0))
        orbit_seq(f, x0, n) = iterate(f, n, x0)
        contraction_iterate_distance(f, c, n, f(x0), x0)
        iterate(f, n, f(x0)).distance(iterate(f, n, x0)) <= c.pow(n) * f(x0).distance(x0)
        orbit_seq(f, x0, n.suc).distance(orbit_seq(f, x0, n)) <= c.pow(n) * f(x0).distance(x0)
        mul_seq(f(x0).distance(x0), c.pow, n) = f(x0).distance(x0) * c.pow(n)
        f(x0).distance(x0) * c.pow(n) = c.pow(n) * f(x0).distance(x0)
        orbit_seq(f, x0, n.suc).distance(orbit_seq(f, x0, n)) <= mul_seq(f(x0).distance(x0), c.pow, n)
    }
}

/// The base case of the orbit polygon bound: the distance from an orbit point
/// to itself is bounded by the empty tail of the dominating partial sums.
theorem orbit_polygon_base[M: MetricSpace](f: M -> M, c: Real, x0: M, k: Nat) {
    is_contraction(f, c) implies
    orbit_seq(f, x0, k).distance(orbit_seq(f, x0, k + Nat.0)) <=
        orbit_partial(f, c, x0, k + Nat.0) - orbit_partial(f, c, x0, k)
} by {
    if is_contraction(f, c) {
        k + Nat.0 = k
        orbit_seq(f, x0, k).distance(orbit_seq(f, x0, k)) = Real.0
        orbit_partial(f, c, x0, k) - orbit_partial(f, c, x0, k) = Real.0
        orbit_seq(f, x0, k).distance(orbit_seq(f, x0, k + Nat.0)) = orbit_seq(f, x0, k).distance(orbit_seq(f, x0, k))
        orbit_partial(f, c, x0, k + Nat.0) = orbit_partial(f, c, x0, k)
        orbit_seq(f, x0, k).distance(orbit_seq(f, x0, k + Nat.0)) <= orbit_partial(f, c, x0, k + Nat.0) - orbit_partial(f, c, x0, k)
    }
}

/// The distance between two orbit points is bounded by the tail of the
/// dominating partial sums.
///
/// For `m <= n`, `d(x_m, x_n)` is at most `Σ_{k=m}^{n-1} c^k · d(f(x0), x0)`,
/// expressed as the difference of the partial sums of the geometric majorant.
theorem orbit_polygon_bound[M: MetricSpace](f: M -> M, c: Real, x0: M, m: Nat, n: Nat) {
    is_contraction(f, c) and m <= n implies
    orbit_seq(f, x0, m).distance(orbit_seq(f, x0, n)) <=
        orbit_partial(f, c, x0, n) - orbit_partial(f, c, x0, m)
} by {
    if is_contraction(f, c) and m <= n {
        define p(j: Nat) -> Bool {
            forall(k: Nat) {
                orbit_seq(f, x0, k).distance(orbit_seq(f, x0, k + j)) <=
                    orbit_partial(f, c, x0, k + j) - orbit_partial(f, c, x0, k)
            }
        }
        forall(k: Nat) {
            orbit_polygon_base(f, c, x0, k)
            orbit_seq(f, x0, k).distance(orbit_seq(f, x0, k + Nat.0)) <= orbit_partial(f, c, x0, k + Nat.0) - orbit_partial(f, c, x0, k)
        }
        p(Nat.0)
        forall(j: Nat) {
            if p(j) {
                forall(k: Nat) {
                    orbit_consecutive_bound(f, c, x0, k + j)
                    orbit_seq(f, x0, (k + j).suc).distance(orbit_seq(f, x0, k + j)) <= mul_seq(f(x0).distance(x0), c.pow, k + j)
                    k + j.suc = (k + j).suc
                    orbit_seq(f, x0, k + j.suc).distance(orbit_seq(f, x0, k + j)) <= mul_seq(f(x0).distance(x0), c.pow, k + j)
                    partial_split_last(mul_seq(f(x0).distance(x0), c.pow), k + j)
                    orbit_partial(f, c, x0, (k + j).suc) = orbit_partial(f, c, x0, k + j) + mul_seq(f(x0).distance(x0), c.pow, k + j)
                    orbit_partial(f, c, x0, k + j.suc) = orbit_partial(f, c, x0, k + j) + mul_seq(f(x0).distance(x0), c.pow, k + j)
                    orbit_partial(f, c, x0, k + j.suc) - orbit_partial(f, c, x0, k + j) = mul_seq(f(x0).distance(x0), c.pow, k + j)
                    orbit_seq(f, x0, k + j.suc).distance(orbit_seq(f, x0, k + j)) <= orbit_partial(f, c, x0, k + j.suc) - orbit_partial(f, c, x0, k + j)
                    orbit_seq(f, x0, k).distance(orbit_seq(f, x0, k + j.suc)) <= orbit_seq(f, x0, k).distance(orbit_seq(f, x0, k + j)) + orbit_seq(f, x0, k + j).distance(orbit_seq(f, x0, k + j.suc))
                    orbit_seq(f, x0, k).distance(orbit_seq(f, x0, k + j)) <= orbit_partial(f, c, x0, k + j) - orbit_partial(f, c, x0, k)
                    add_le_add(
                        orbit_seq(f, x0, k).distance(orbit_seq(f, x0, k + j)),
                        orbit_partial(f, c, x0, k + j) - orbit_partial(f, c, x0, k),
                        orbit_seq(f, x0, k + j).distance(orbit_seq(f, x0, k + j.suc)),
                        mul_seq(f(x0).distance(x0), c.pow, k + j)
                    )
                    orbit_seq(f, x0, k).distance(orbit_seq(f, x0, k + j)) + orbit_seq(f, x0, k + j).distance(orbit_seq(f, x0, k + j.suc)) <= orbit_partial(f, c, x0, k + j) - orbit_partial(f, c, x0, k) + mul_seq(f(x0).distance(x0), c.pow, k + j)
                    orbit_partial(f, c, x0, k + j) - orbit_partial(f, c, x0, k) + mul_seq(f(x0).distance(x0), c.pow, k + j) = orbit_partial(f, c, x0, k + j.suc) - orbit_partial(f, c, x0, k)
                    orbit_seq(f, x0, k).distance(orbit_seq(f, x0, k + j)) + orbit_seq(f, x0, k + j).distance(orbit_seq(f, x0, k + j.suc)) <= orbit_partial(f, c, x0, k + j.suc) - orbit_partial(f, c, x0, k)
                    lte_trans(orbit_seq(f, x0, k).distance(orbit_seq(f, x0, k + j.suc)), orbit_seq(f, x0, k).distance(orbit_seq(f, x0, k + j)) + orbit_seq(f, x0, k + j).distance(orbit_seq(f, x0, k + j.suc)), orbit_partial(f, c, x0, k + j.suc) - orbit_partial(f, c, x0, k))
                    orbit_seq(f, x0, k).distance(orbit_seq(f, x0, k + j.suc)) <= orbit_partial(f, c, x0, k + j.suc) - orbit_partial(f, c, x0, k)
                }
                p(j.suc)
            }
        }
        p(Nat.0) and forall(j: Nat) {
            p(j) implies p(j.suc)
        }
        Nat.induction(p)
        p(n - m)
        orbit_seq(f, x0, m).distance(orbit_seq(f, x0, m + (n - m))) <= orbit_partial(f, c, x0, m + (n - m)) - orbit_partial(f, c, x0, m)
        add_sub(n, m)
        n - m + m = n
        m + (n - m) = n
        orbit_partial(f, c, x0, m + (n - m)) = orbit_partial(f, c, x0, n)
        orbit_seq(f, x0, m).distance(orbit_seq(f, x0, n)) <= orbit_partial(f, c, x0, n) - orbit_partial(f, c, x0, m)
    }
}

/// If the partial sums of the geometric majorant are close at `i` and `j`,
/// then the corresponding orbit points are close.
///
/// This is the two-case (ordered indices) core of the Cauchy proof: the
/// polygon bound transfers the closeness of the partial sums to the orbit.
theorem orbit_pair_bound[M: MetricSpace](f: M -> M, c: Real, x0: M, eps: Real, n: Nat, i: Nat, j: Nat) {
    is_contraction(f, c)
    and (forall(a: Nat, b: Nat) {
        n <= a and n <= b implies
            partial(mul_seq(f(x0).distance(x0), c.pow), a).is_close(partial(mul_seq(f(x0).distance(x0), c.pow), b), eps)
    })
    and n <= i and n <= j
    implies orbit_seq(f, x0, i).distance(orbit_seq(f, x0, j)) < eps
} by {
    if is_contraction(f, c)
        and (forall(a: Nat, b: Nat) {
            n <= a and n <= b implies
                partial(mul_seq(f(x0).distance(x0), c.pow), a).is_close(partial(mul_seq(f(x0).distance(x0), c.pow), b), eps)
        })
        and n <= i and n <= j {
        if i <= j {
            orbit_polygon_bound(f, c, x0, i, j)
            orbit_seq(f, x0, i).distance(orbit_seq(f, x0, j)) <= orbit_partial(f, c, x0, j) - orbit_partial(f, c, x0, i)
            partial(mul_seq(f(x0).distance(x0), c.pow), i).is_close(partial(mul_seq(f(x0).distance(x0), c.pow), j), eps)
            (partial(mul_seq(f(x0).distance(x0), c.pow), j) - partial(mul_seq(f(x0).distance(x0), c.pow), i)).abs < eps
            orbit_partial(f, c, x0, j) - orbit_partial(f, c, x0, i) = partial(mul_seq(f(x0).distance(x0), c.pow), j) - partial(mul_seq(f(x0).distance(x0), c.pow), i)
            (orbit_partial(f, c, x0, j) - orbit_partial(f, c, x0, i)).abs < eps
            lte_abs(orbit_partial(f, c, x0, j) - orbit_partial(f, c, x0, i))
            orbit_partial(f, c, x0, j) - orbit_partial(f, c, x0, i) <= (orbit_partial(f, c, x0, j) - orbit_partial(f, c, x0, i)).abs
            lte_trans(orbit_seq(f, x0, i).distance(orbit_seq(f, x0, j)), orbit_partial(f, c, x0, j) - orbit_partial(f, c, x0, i), (orbit_partial(f, c, x0, j) - orbit_partial(f, c, x0, i)).abs)
            orbit_seq(f, x0, i).distance(orbit_seq(f, x0, j)) <= (orbit_partial(f, c, x0, j) - orbit_partial(f, c, x0, i)).abs
            lte_lt_trans(orbit_seq(f, x0, i).distance(orbit_seq(f, x0, j)), (orbit_partial(f, c, x0, j) - orbit_partial(f, c, x0, i)).abs, eps)
            orbit_seq(f, x0, i).distance(orbit_seq(f, x0, j)) < eps
        }
        if not i <= j {
            not_lte_imp_gt(i, j)
            i > j
            lt_imp_lte(j, i)
            j <= i
            orbit_polygon_bound(f, c, x0, j, i)
            orbit_seq(f, x0, j).distance(orbit_seq(f, x0, i)) <= orbit_partial(f, c, x0, i) - orbit_partial(f, c, x0, j)
            partial(mul_seq(f(x0).distance(x0), c.pow), j).is_close(partial(mul_seq(f(x0).distance(x0), c.pow), i), eps)
            (partial(mul_seq(f(x0).distance(x0), c.pow), i) - partial(mul_seq(f(x0).distance(x0), c.pow), j)).abs < eps
            orbit_partial(f, c, x0, i) - orbit_partial(f, c, x0, j) = partial(mul_seq(f(x0).distance(x0), c.pow), i) - partial(mul_seq(f(x0).distance(x0), c.pow), j)
            (orbit_partial(f, c, x0, i) - orbit_partial(f, c, x0, j)).abs < eps
            lte_abs(orbit_partial(f, c, x0, i) - orbit_partial(f, c, x0, j))
            orbit_partial(f, c, x0, i) - orbit_partial(f, c, x0, j) <= (orbit_partial(f, c, x0, i) - orbit_partial(f, c, x0, j)).abs
            lte_trans(orbit_seq(f, x0, j).distance(orbit_seq(f, x0, i)), orbit_partial(f, c, x0, i) - orbit_partial(f, c, x0, j), (orbit_partial(f, c, x0, i) - orbit_partial(f, c, x0, j)).abs)
            orbit_seq(f, x0, j).distance(orbit_seq(f, x0, i)) <= (orbit_partial(f, c, x0, i) - orbit_partial(f, c, x0, j)).abs
            lte_lt_trans(orbit_seq(f, x0, j).distance(orbit_seq(f, x0, i)), (orbit_partial(f, c, x0, i) - orbit_partial(f, c, x0, j)).abs, eps)
            orbit_seq(f, x0, j).distance(orbit_seq(f, x0, i)) < eps
            orbit_seq(f, x0, i).distance(orbit_seq(f, x0, j)) = orbit_seq(f, x0, j).distance(orbit_seq(f, x0, i))
            orbit_seq(f, x0, i).distance(orbit_seq(f, x0, j)) < eps
        }
        orbit_seq(f, x0, i).distance(orbit_seq(f, x0, j)) < eps
    }
}

/// The orbit of a contraction is a Cauchy sequence.
theorem orbit_cauchy_metric[M: MetricSpace](f: M -> M, c: Real, x0: M) {
    is_contraction(f, c) implies is_cauchy_metric(orbit_seq(f, x0))
} by {
    if is_contraction(f, c) {
        forall(eps: Real) {
            if eps.is_positive {
                contraction_constant_nonneg(f, c)
                Real.0 <= c
                abs_of_nonneg(c)
                c.abs = c
                contraction_constant_lt_one(f, c)
                c < Real.1
                c.abs < Real.1
                geom_converges(c)
                converges(partial(c.pow))
                converges_mul_seq(f(x0).distance(x0), partial(c.pow))
                converges(mul_seq(f(x0).distance(x0), partial(c.pow)))
                partial_mul_seq(f(x0).distance(x0), c.pow, Nat.0)
                partial(mul_seq(f(x0).distance(x0), c.pow), Nat.0) = mul_seq(f(x0).distance(x0), partial(c.pow), Nat.0)
                function_extensionality[Nat, Real](partial(mul_seq(f(x0).distance(x0), c.pow)), mul_seq(f(x0).distance(x0), partial(c.pow)))
                partial(mul_seq(f(x0).distance(x0), c.pow)) = mul_seq(f(x0).distance(x0), partial(c.pow))
                converges(partial(mul_seq(f(x0).distance(x0), c.pow)))
                converges(partial(mul_seq(f(x0).distance(x0), c.pow))) = forall(e: Real) {
                    e.is_positive implies exists(n: Nat) {
                        cauchy_bound(partial(mul_seq(f(x0).distance(x0), c.pow)), n, e)
                    }
                }
                exists(n: Nat) {
                    cauchy_bound(partial(mul_seq(f(x0).distance(x0), c.pow)), n, eps)
                }
                let n: Nat satisfy {
                    cauchy_bound(partial(mul_seq(f(x0).distance(x0), c.pow)), n, eps)
                }
                cauchy_bound(partial(mul_seq(f(x0).distance(x0), c.pow)), n, eps)
                cauchy_bound(partial(mul_seq(f(x0).distance(x0), c.pow)), n, eps) = forall(i: Nat, j: Nat) {
                    n <= i and n <= j implies partial(mul_seq(f(x0).distance(x0), c.pow), i).is_close(partial(mul_seq(f(x0).distance(x0), c.pow), j), eps)
                }
                forall(i: Nat, j: Nat) {
                    if n <= i and n <= j {
                        orbit_pair_bound(f, c, x0, eps, n, i, j)
                        orbit_seq(f, x0, i).distance(orbit_seq(f, x0, j)) < eps
                    }
                }
                cauchy_metric_bound(orbit_seq(f, x0), n, eps) = forall(i: Nat, j: Nat) {
                    n <= i and n <= j implies orbit_seq(f, x0, i).distance(orbit_seq(f, x0, j)) < eps
                }
                forall(i: Nat, j: Nat) {
                    n <= i and n <= j implies orbit_seq(f, x0, i).distance(orbit_seq(f, x0, j)) < eps
                }
                cauchy_metric_bound_from_distances(orbit_seq(f, x0), n, eps)
                cauchy_metric_bound(orbit_seq(f, x0), n, eps)
                exists(m: Nat) {
                    cauchy_metric_bound(orbit_seq(f, x0), m, eps)
                }
            }
        }
    }
}

/// A real that is strictly below every positive real is at most zero.
theorem lt_all_positive_eps_imp_lte_zero(x: Real) {
    (forall(eps: Real) { eps.is_positive implies x < eps })
    implies x <= Real.0
} by {
    if forall(eps: Real) { eps.is_positive implies x < eps } {
        if not x <= Real.0 {
            not_lte_imp_gt(x, Real.0)
            Real.0 < x
            gt_zero_imp_pos(x)
            x.is_positive
            x.is_positive implies x < x
            x < x
            not_lt_self(x)
            false
        }
        x <= Real.0
    }
}

/// The orbit of a contraction of a complete metric space converges to a fixed
/// point.
///
/// This is the convergence half of the Banach fixed point theorem: the orbit
/// of any starting point is Cauchy (the geometric bound on consecutive
/// distances), completeness supplies a limit, and the contraction inequality
/// forces the limit to be a fixed point.
theorem banach_contraction_orbit_converges[M: CompleteMetricSpace](f: M -> M, c: Real, x0: M) {
    is_contraction(f, c) implies exists(a: M) {
        is_fixed_point(f, a) and tendsto_metric(orbit_seq(f, x0), a)
    }
} by {
    if is_contraction(f, c) {
        orbit_cauchy_metric(f, c, x0)
        is_cauchy_metric(orbit_seq(f, x0))
        exists(a: M) {
            tendsto_metric(orbit_seq(f, x0), a)
        }
        let a: M satisfy {
            tendsto_metric(orbit_seq(f, x0), a)
        }
        tendsto_metric(orbit_seq(f, x0), a)
        forall(eps: Real) {
            if eps.is_positive {
                let half: Real satisfy {
                    half.is_positive and half + half < eps
                }
                tendsto_metric_tail_bound(orbit_seq(f, x0), a, half)
                let n: Nat satisfy {
                    metric_tail_bound(orbit_seq(f, x0), a, n, half)
                }
                metric_tail_bound_at(orbit_seq(f, x0), a, n, half, n)
                orbit_seq(f, x0, n).distance(a) < half
                n <= n.suc
                metric_tail_bound_at(orbit_seq(f, x0), a, n, half, n.suc)
                orbit_seq(f, x0, n.suc).distance(a) < half
                orbit_seq(f, x0, n.suc) = f(orbit_seq(f, x0, n))
                f(orbit_seq(f, x0, n)).distance(a) < half
                contraction_distance(f, c, a, orbit_seq(f, x0, n))
                f(a).distance(f(orbit_seq(f, x0, n))) <= c * a.distance(orbit_seq(f, x0, n))
                a.distance(orbit_seq(f, x0, n)) = orbit_seq(f, x0, n).distance(a)
                a.distance(orbit_seq(f, x0, n)) < half
                a.distance(orbit_seq(f, x0, n)) <= half
                contraction_constant_nonneg(f, c)
                Real.0 <= c
                mul_le_mul_of_nonneg_left(a.distance(orbit_seq(f, x0, n)), half, c)
                c * a.distance(orbit_seq(f, x0, n)) <= c * half
                lte_trans(f(a).distance(f(orbit_seq(f, x0, n))), c * a.distance(orbit_seq(f, x0, n)), c * half)
                f(a).distance(f(orbit_seq(f, x0, n))) <= c * half
                mul_lt_mul_of_pos_left(c, Real.1, half)
                c * half < Real.1 * half
                Real.1 * half = half
                c * half < half
                f(a).distance(f(orbit_seq(f, x0, n))) + f(orbit_seq(f, x0, n)).distance(a) <= c * half + f(orbit_seq(f, x0, n)).distance(a)
                add_lt_add_left(f(orbit_seq(f, x0, n)).distance(a), half, c * half)
                c * half + f(orbit_seq(f, x0, n)).distance(a) < c * half + half
                c * half + half < half + half
                c * half + f(orbit_seq(f, x0, n)).distance(a) < half + half
                lte_lt_trans(f(a).distance(f(orbit_seq(f, x0, n))) + f(orbit_seq(f, x0, n)).distance(a), c * half + f(orbit_seq(f, x0, n)).distance(a), half + half)
                f(a).distance(f(orbit_seq(f, x0, n))) + f(orbit_seq(f, x0, n)).distance(a) < half + half
                f(a).distance(a) <= f(a).distance(f(orbit_seq(f, x0, n))) + f(orbit_seq(f, x0, n)).distance(a)
                lte_lt_trans(f(a).distance(a), f(a).distance(f(orbit_seq(f, x0, n))) + f(orbit_seq(f, x0, n)).distance(a), half + half)
                f(a).distance(a) < half + half
                half + half < eps
                lt_trans(f(a).distance(a), half + half, eps)
                f(a).distance(a) < eps
            }
        }
        lt_all_positive_eps_imp_lte_zero(f(a).distance(a))
        f(a).distance(a) <= Real.0
        distance_non_negative(f(a), a)
        not f(a).distance(a).is_negative
        Real.0 <= f(a).distance(a)
        f(a).distance(a) = Real.0
        f(a) = a
        is_fixed_point(f, a)
        is_fixed_point(f, a) and tendsto_metric(orbit_seq(f, x0), a)
        exists(b: M) {
            is_fixed_point(f, b) and tendsto_metric(orbit_seq(f, x0), b)
        }
    }
}

/// A contraction of a complete metric space has a fixed point.
theorem banach_contraction_fixed_point_exists[M: CompleteMetricSpace](f: M -> M, c: Real, x0: M) {
    is_contraction(f, c) implies exists(a: M) {
        is_fixed_point(f, a)
    }
} by {
    if is_contraction(f, c) {
        banach_contraction_orbit_converges(f, c, x0)
        exists(a: M) {
            is_fixed_point(f, a) and tendsto_metric(orbit_seq(f, x0), a)
        }
        let a: M satisfy {
            is_fixed_point(f, a) and tendsto_metric(orbit_seq(f, x0), a)
        }
        is_fixed_point(f, a)
        exists(b: M) {
            is_fixed_point(f, b)
        }
    }
}

/// Two fixed points of a contraction coincide.
theorem banach_contraction_fixed_point_unique[M: MetricSpace](f: M -> M, c: Real, x: M, y: M) {
    is_contraction(f, c) and is_fixed_point(f, x) and is_fixed_point(f, y) implies x = y
} by {
    if is_contraction(f, c) and is_fixed_point(f, x) and is_fixed_point(f, y) {
        is_fixed_point(f, x) = (f(x) = x)
        is_fixed_point(f, y) = (f(y) = y)
        f(x) = x
        f(y) = y
        x.distance(y) = f(x).distance(f(y))
        contraction_distance(f, c, x, y)
        f(x).distance(f(y)) <= c * x.distance(y)
        x.distance(y) <= c * x.distance(y)
        distance_non_negative(x, y)
        not x.distance(y).is_negative
        Real.0 <= x.distance(y)
        eq_or_lt_of_lte(Real.0, x.distance(y))
        Real.0 = x.distance(y) or Real.0 < x.distance(y)
        if x.distance(y) = Real.0 {
            x.distance(y) = Real.0
        }
        if not x.distance(y) = Real.0 {
            Real.0 < x.distance(y)
            contraction_constant_lt_one(f, c)
            c < Real.1
            mul_lt_mul_of_pos_left(c, Real.1, x.distance(y))
            c * x.distance(y) < Real.1 * x.distance(y)
            Real.1 * x.distance(y) = x.distance(y)
            c * x.distance(y) < x.distance(y)
            x.distance(y) <= c * x.distance(y)
            lte_lt_trans(x.distance(y), c * x.distance(y), x.distance(y))
            x.distance(y) < x.distance(y)
            not_lt_self(x.distance(y))
            false
        }
        x.distance(y) = Real.0
        x = y
    }
}

/// The Banach fixed point theorem: a contraction of a complete metric space
/// has a unique fixed point.
theorem banach_contraction_fixed_point[M: CompleteMetricSpace](f: M -> M, c: Real, x0: M) {
    is_contraction(f, c) implies exists(a: M) {
        is_fixed_point(f, a) and forall(y: M) {
            is_fixed_point(f, y) implies y = a
        }
    }
} by {
    if is_contraction(f, c) {
        banach_contraction_fixed_point_exists(f, c, x0)
        exists(a: M) {
            is_fixed_point(f, a)
        }
        let a: M satisfy {
            is_fixed_point(f, a)
        }
        is_fixed_point(f, a)
        forall(y: M) {
            if is_fixed_point(f, y) {
                banach_contraction_fixed_point_unique(f, c, a, y)
                a = y
                y = a
            }
        }
        is_fixed_point(f, a) and forall(y: M) {
            is_fixed_point(f, y) implies y = a
        }
        exists(b: M) {
            is_fixed_point(f, b) and forall(y: M) {
                is_fixed_point(f, y) implies y = b
            }
        }
    }
}

/// The Banach fixed point theorem on the real line.
///
/// A self-map of the reals that shrinks absolute distances by a factor `c` in
/// `[0, 1)` has a unique fixed point; the reals are complete, so this is the
/// metric-space theorem specialized to `Real`.
theorem real_banach_fixed_point(f: Real -> Real, c: Real, x0: Real) {
    Real.0 <= c
    and c < Real.1
    and (forall(x: Real, y: Real) {
        (f(x) - f(y)).abs <= c * (x - y).abs
    })
    implies exists(a: Real) {
        f(a) = a
    }
} by {
    if Real.0 <= c and c < Real.1 and (forall(x: Real, y: Real) {
        (f(x) - f(y)).abs <= c * (x - y).abs
    }) {
        forall(x: Real, y: Real) {
            f(x).distance(f(y)) = real_distance(f(x), f(y))
            real_distance(f(x), f(y)) = (f(x) - f(y)).abs
            x.distance(y) = real_distance(x, y)
            real_distance(x, y) = (x - y).abs
            (f(x) - f(y)).abs <= c * (x - y).abs
            f(x).distance(f(y)) <= c * x.distance(y)
        }
        is_contraction[Real](f, c)
        banach_contraction_fixed_point[Real](f, c, x0)
        exists(a: Real) {
            is_fixed_point(f, a) and forall(y: Real) {
                is_fixed_point(f, y) implies y = a
            }
        }
        let a: Real satisfy {
            is_fixed_point(f, a) and forall(y: Real) {
                is_fixed_point(f, y) implies y = a
            }
        }
        is_fixed_point(f, a)
        is_fixed_point(f, a) = (f(a) = a)
        f(a) = a
        exists(b: Real) {
            f(b) = b
        }
    }
}
