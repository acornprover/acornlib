from data.basic.set import Set, set_preimage
from analysis.metric.metric_space import MetricSpace, is_open_metric, is_closed_metric,
    open_ball, open_ball_is_open_metric
from analysis.topology.topological_space import TopologicalSpace, is_closed, is_continuous
from analysis.topology.continuous_preimage import continuous_open_preimage
from real import Real

/// A metric space whose topological open sets are exactly its metric-open sets.
///
/// This packages the compatibility needed to use the generic topology API on a
/// metric carrier without changing the existing `MetricSpace` class globally.
typeclass M: MetricTopologicalSpace extends MetricSpace, TopologicalSpace {
    /// Compatibility axiom: topological openness agrees with metric openness.
    topological_open_eq_metric_open(s: Set[M]) {
        M.is_open(s) = is_open_metric(s)
    }
}

/// In a compatible metric topological space, topological openness is metric openness.
theorem topological_open_eq_metric_open[M: MetricTopologicalSpace](s: Set[M]) {
    M.is_open(s) = is_open_metric(s)
}

/// Metric-open sets are open for the generic `TopologicalSpace` API.
theorem metric_open_is_topological_open[M: MetricTopologicalSpace](s: Set[M]) {
    is_open_metric(s) implies M.is_open(s)
} by {
    if is_open_metric(s) {
        topological_open_eq_metric_open(s)
        M.is_open(s)
    }
}

/// Generic topological open sets are metric-open sets.
theorem topological_open_is_metric_open[M: MetricTopologicalSpace](s: Set[M]) {
    M.is_open(s) implies is_open_metric(s)
} by {
    if M.is_open(s) {
        topological_open_eq_metric_open(s)
        is_open_metric(s)
    }
}

/// Metric-closed sets are exactly closed sets in the generic topological sense.
theorem metric_closed_eq_topological_closed[M: MetricTopologicalSpace](s: Set[M]) {
    is_closed_metric(s) = is_closed(s)
} by {
    topological_open_eq_metric_open(s.c)
}

/// Metric-closed sets are closed for the generic `TopologicalSpace` API.
theorem metric_closed_is_topological_closed[M: MetricTopologicalSpace](s: Set[M]) {
    is_closed_metric(s) implies is_closed(s)
} by {
    if is_closed_metric(s) {
        metric_closed_eq_topological_closed(s)
        is_closed(s)
    }
}

/// Generic closed sets are metric-closed sets.
theorem topological_closed_is_metric_closed[M: MetricTopologicalSpace](s: Set[M]) {
    is_closed(s) implies is_closed_metric(s)
} by {
    if is_closed(s) {
        metric_closed_eq_topological_closed(s)
        is_closed_metric(s)
    }
}

/// Every metric open ball is topologically open.
theorem open_ball_is_topological_open[M: MetricTopologicalSpace](x: M, r: Real) {
    M.is_open(open_ball(x, r))
} by {
    open_ball_is_open_metric(x, r)
    metric_open_is_topological_open(open_ball(x, r))
}

/// A continuous map into a compatible metric topology has topologically-open preimage
/// for every metric-open target set.
theorem continuous_preimage_metric_open[M: TopologicalSpace, N: MetricTopologicalSpace](f: M -> N, v: Set[N]) {
    is_continuous(f) and is_open_metric(v) implies M.is_open(set_preimage(f, v))
} by {
    if is_continuous(f) and is_open_metric(v) {
        metric_open_is_topological_open(v)
        continuous_open_preimage(f, v)
        M.is_open(set_preimage(f, v))
    }
}

/// A continuous map between compatible metric topologies has metric-open preimage
/// for every metric-open target set.
theorem continuous_preimage_metric_open_is_metric_open[M: MetricTopologicalSpace, N: MetricTopologicalSpace](f: M -> N, v: Set[N]) {
    is_continuous(f) and is_open_metric(v) implies is_open_metric(set_preimage(f, v))
} by {
    if is_continuous(f) and is_open_metric(v) {
        continuous_preimage_metric_open(f, v)
        topological_open_is_metric_open(set_preimage(f, v))
        is_open_metric(set_preimage(f, v))
    }
}
