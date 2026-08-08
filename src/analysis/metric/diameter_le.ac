from real import Real
from real import lte_trans
from real import is_set_supremum, is_set_upper_bound
from data.basic.set import Set
from analysis.metric.metric_space import MetricSpace, closed_ball, open_ball, is_bounded
from analysis.metric.metric_space import has_diameter, is_diameter, diameter, distance_set, diameter_is_diameter,
    distance_le_diameter_fn, diameter_le_of_subset
from analysis.metric.metric_space import bounded_distance_upper_bound, closed_ball_is_bounded, open_ball_is_bounded
from analysis.metric.metric_space import center_in_closed_ball, center_in_open_ball, open_ball_subset_closed_ball

/// An upper bound dominates each set element.
theorem upper_bound_apply(s: Set[Real], u: Real, z: Real) {
    is_set_upper_bound(s, u) and s.contains(z) implies z <= u
} by {
    if is_set_upper_bound(s, u) and s.contains(z) {
        is_set_upper_bound(s, u) = forall(y: Real) {
            s.contains(y) implies y <= u
        }
        z <= u
    }
}

/// If `u` is an upper bound of `s` and `u <= b`, then `b` is also an upper bound.
theorem upper_bound_weaken(s: Set[Real], u: Real, b: Real) {
    is_set_upper_bound(s, u) and u <= b implies is_set_upper_bound(s, b)
} by {
    if is_set_upper_bound(s, u) and u <= b {
        forall(z: Real) {
            if s.contains(z) {
                upper_bound_apply(s, u, z)
                lte_trans(z, u, b)
                z <= b
            }
        }
        is_set_upper_bound(s, b)
    }
}

/// The supremum is the least upper bound.
theorem supremum_le_of_upper_bound(s: Set[Real], sup_s: Real, b: Real) {
    is_set_supremum(s, sup_s) and is_set_upper_bound(s, b) implies sup_s <= b
} by {
    if is_set_supremum(s, sup_s) and is_set_upper_bound(s, b) {
        let least: Bool = forall(c: Real) {
            is_set_upper_bound(s, c) implies sup_s <= c
        }
        is_set_supremum(s, sup_s) = (is_set_upper_bound(s, sup_s) and least)
        sup_s <= b
    }
}

/// If `b` is an upper bound of `distance_set(s)` and `s` has a diameter, then
/// `diameter(s) <= b`.
theorem diameter_le_of_upper_bound[M: MetricSpace](s: Set[M], b: Real) {
    has_diameter(s) and is_set_upper_bound(distance_set(s), b)
    implies diameter(s) <= b
} by {
    if has_diameter(s) and is_set_upper_bound(distance_set(s), b) {
        let ds: Set[Real] = distance_set(s)
        diameter_is_diameter(s)
        is_set_supremum(ds, diameter(s))
        supremum_le_of_upper_bound(ds, diameter(s), b)
        diameter(s) <= b
    }
}

/// Characterization of `diameter(s) <= b` via the bound on `distance_set(s)`:
/// when `s` has a finite diameter, `diameter(s) <= b` iff `b` is an upper bound
/// of all pairwise distances in `s`.
theorem diameter_le[M: MetricSpace](s: Set[M], b: Real) {
    has_diameter(s) implies (
        diameter(s) <= b iff is_set_upper_bound(distance_set(s), b)
    )
} by {
    if has_diameter(s) {
        let ds: Set[Real] = distance_set(s)
        diameter_is_diameter(s)
        is_set_supremum(ds, diameter(s))
        let least: Bool = forall(c: Real) {
            is_set_upper_bound(ds, c) implies diameter(s) <= c
        }
        is_set_supremum(ds, diameter(s)) = (is_set_upper_bound(ds, diameter(s)) and least)
        if diameter(s) <= b {
            upper_bound_weaken(ds, diameter(s), b)
            is_set_upper_bound(distance_set(s), b)
        }
        if is_set_upper_bound(distance_set(s), b) {
            diameter_le_of_upper_bound(s, b)
            diameter(s) <= b
        }
        diameter(s) <= b iff is_set_upper_bound(distance_set(s), b)
    }
}

/// A set with a diameter and contained in a closed ball has diameter at most twice the radius.
theorem diameter_le_of_subset_closed_ball[M: MetricSpace](s: Set[M], c: M, r: Real) {
    has_diameter(s) and s.subset(closed_ball(c, r)) implies diameter(s) <= r + r
} by {
    if has_diameter(s) and s.subset(closed_ball(c, r)) {
        bounded_distance_upper_bound(s, c, r)
        diameter_le_of_upper_bound(s, r + r)
        diameter(s) <= r + r
    }
}

/// A nonempty bounded set has a diameter.
theorem has_diameter_of_bounded_contains[M: MetricSpace](s: Set[M], x: M) {
    is_bounded(s) and s.contains(x) implies has_diameter(s)
} by {
    if is_bounded(s) and s.contains(x) {
        has_diameter(s)
    }
}

/// A closed ball with nonnegative radius has diameter at most twice its radius.
theorem closed_ball_diameter_le[M: MetricSpace](c: M, r: Real) {
    not r.is_negative implies diameter(closed_ball(c, r)) <= r + r
} by {
    if not r.is_negative {
        closed_ball_is_bounded(c, r)
        center_in_closed_ball(c, r)
        has_diameter_of_bounded_contains(closed_ball(c, r), c)
        closed_ball(c, r).subset(closed_ball(c, r))
        diameter_le_of_subset_closed_ball(closed_ball(c, r), c, r)
        diameter(closed_ball(c, r)) <= r + r
    }
}

/// An open ball with positive radius has diameter at most twice its radius.
theorem open_ball_diameter_le[M: MetricSpace](c: M, r: Real) {
    r.is_positive implies diameter(open_ball(c, r)) <= r + r
} by {
    if r.is_positive {
        open_ball_is_bounded(c, r)
        center_in_open_ball(c, r)
        has_diameter_of_bounded_contains(open_ball(c, r), c)
        open_ball_subset_closed_ball(c, r)
        diameter_le_of_subset_closed_ball(open_ball(c, r), c, r)
        diameter(open_ball(c, r)) <= r + r
    }
}

/// If the diameter is bounded by `b`, then every distance in the set is bounded by `b`.
theorem distance_le_of_diameter_le[M: MetricSpace](s: Set[M], b: Real, x: M, y: M) {
    has_diameter(s) and s.contains(x) and s.contains(y) and diameter(s) <= b implies
    x.distance(y) <= b
} by {
    if has_diameter(s) and s.contains(x) and s.contains(y) and diameter(s) <= b {
        distance_le_diameter_fn(s, x, y)
        lte_trans(x.distance(y), diameter(s), b)
        x.distance(y) <= b
    }
}

/// A subset inherits any upper bound on the diameter of the larger set.
theorem diameter_le_of_subset_with_bound[M: MetricSpace](s: Set[M], t: Set[M], b: Real) {
    s.subset(t) and has_diameter(s) and has_diameter(t) and diameter(t) <= b implies
    diameter(s) <= b
} by {
    if s.subset(t) and has_diameter(s) and has_diameter(t) and diameter(t) <= b {
        diameter_le_of_subset(s, t)
        lte_trans(diameter(s), diameter(t), b)
        diameter(s) <= b
    }
}
