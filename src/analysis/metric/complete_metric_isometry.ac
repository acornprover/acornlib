/// Completeness transport along surjective isometries of metric spaces.

from nat import Nat
from data.basic.functions import compose, function_extensionality, function_eq_apply,
    function_eq_transport_predicate, is_surjective_fn, surjective_fn_has_preimage,
    is_right_inverse_fn, right_inverse_fn_apply
from analysis.metric.metric_space import MetricSpace, is_isometry, isometry_distance,
    isometry_preserves_cauchy_metric, isometry_preserves_tendsto_metric,
    is_cauchy_metric, tendsto_metric
from analysis.metric.complete_metric_space import CompleteMetricSpace

/// A pointwise lift of a target sequence along a map.
define is_sequence_lift[M, N](f: M -> N, p: Nat -> M, q: Nat -> N) -> Bool {
    compose(f, p) = q
}

/// A pointwise equality proof packages a sequence lift.
theorem sequence_lift_of_apply[M, N](f: M -> N, p: Nat -> M, q: Nat -> N) {
    (forall(n: Nat) { f(p(n)) = q(n) }) implies is_sequence_lift(f, p, q)
} by {
    if forall(n: Nat) { f(p(n)) = q(n) } {
        forall(n: Nat) {
            compose(f, p)(n) = q(n)
        }
        function_extensionality(compose(f, p), q)
        is_sequence_lift(f, p, q)
    }
}

/// A pointwise lift evaluates to the target sequence at each index.
theorem sequence_lift_apply[M, N](f: M -> N, p: Nat -> M, q: Nat -> N, n: Nat) {
    is_sequence_lift(f, p, q) implies f(p(n)) = q(n)
} by {
    if is_sequence_lift(f, p, q) {
        function_eq_apply(compose(f, p), q, n)
        f(p(n)) = q(n)
    }
}

/// A surjective map has a chosen right inverse.
theorem surjective_has_right_inverse[M, N](f: M -> N) {
    is_surjective_fn(f) implies exists(g: N -> M) {
        is_right_inverse_fn(f, g)
    }
} by {
    if is_surjective_fn(f) {
        forall(y: N) {
            surjective_fn_has_preimage(f, y)
            exists(x: M) {
                f(x) = y
            }
        }
        let g(y: N) -> result: M satisfy {
            f(result) = y
        }
        forall(y: N) {
            f(g(y)) = y
        }
        is_right_inverse_fn(f, g) = forall(y: N) {
            f(g(y)) = y
        }
        exists(h: N -> M) {
            is_right_inverse_fn(f, h)
        }
    }
}

/// A right inverse lifts every target sequence by composition.
theorem right_inverse_sequence_lift[M, N](f: M -> N, g: N -> M, q: Nat -> N) {
    is_right_inverse_fn(f, g) implies is_sequence_lift(f, compose(g, q), q)
} by {
    if is_right_inverse_fn(f, g) {
        forall(n: Nat) {
            right_inverse_fn_apply(f, g, q(n))
            f(compose(g, q)(n)) = q(n)
        }
        sequence_lift_of_apply(f, compose(g, q), q)
        is_sequence_lift(f, compose(g, q), q)
    }
}

/// A surjective map admits a pointwise lift of every sequence.
theorem surjective_sequence_lift[M, N](f: M -> N, q: Nat -> N) {
    is_surjective_fn(f) implies exists(p: Nat -> M) {
        is_sequence_lift(f, p, q)
    }
} by {
    if is_surjective_fn(f) {
        surjective_has_right_inverse(f)
        let g: N -> M satisfy {
            is_right_inverse_fn(f, g)
        }
        right_inverse_sequence_lift(f, g, q)
        exists(p: Nat -> M) {
            is_sequence_lift(f, p, q)
        }
    }
}

/// A right inverse of an isometry is itself an isometry.
theorem right_inverse_of_isometry_is_isometry[M: MetricSpace, N: MetricSpace](
    f: M -> N, g: N -> M
) {
    is_isometry(f) and is_right_inverse_fn(f, g) implies is_isometry(g)
} by {
    if is_isometry(f) and is_right_inverse_fn(f, g) {
        forall(x: N, y: N) {
            right_inverse_fn_apply(f, g, x)
            right_inverse_fn_apply(f, g, y)
            isometry_distance(f, g(x), g(y))
            x.distance(y) = f(g(x)).distance(f(g(y)))
            g(x).distance(g(y)) = x.distance(y)
        }
        is_isometry(g) = forall(x: N, y: N) {
            g(x).distance(g(y)) = x.distance(y)
        }
        if not is_isometry(g) {
            let (x0: N, y0: N) satisfy {
                g(x0).distance(g(y0)) != x0.distance(y0)
            }
            right_inverse_fn_apply(f, g, x0)
            right_inverse_fn_apply(f, g, y0)
            isometry_distance(f, g(x0), g(y0))
            false
        }
        is_isometry(g)
    }
}

/// A right inverse of an isometry sends Cauchy target sequences to Cauchy source sequences.
theorem right_inverse_preserves_cauchy_metric[M: MetricSpace, N: MetricSpace](
    f: M -> N, g: N -> M, q: Nat -> N
) {
    is_isometry(f) and is_right_inverse_fn(f, g) and is_cauchy_metric(q)
        implies is_cauchy_metric(compose(g, q))
} by {
    if is_isometry(f) and is_right_inverse_fn(f, g) and is_cauchy_metric(q) {
        right_inverse_of_isometry_is_isometry(f, g)
        isometry_preserves_cauchy_metric(g, q)
        is_cauchy_metric(compose(g, q))
    }
}

/// A right inverse cancels a map after composition with a target sequence.
theorem right_inverse_compose_sequence[M, N](f: M -> N, g: N -> M, q: Nat -> N) {
    is_right_inverse_fn(f, g) implies compose(f, compose(g, q)) = q
} by {
    if is_right_inverse_fn(f, g) {
        forall(n: Nat) {
            right_inverse_fn_apply(f, g, q(n))
            f(g(q(n))) = q(n)
            compose(f, compose(g, q))(n) = q(n)
        }
        function_extensionality(compose(f, compose(g, q)), q)
        compose(f, compose(g, q)) = q
    }
}

/// A pointwise lift lets metric convergence of the composed source sequence descend
/// to metric convergence of the original target sequence.
theorem sequence_lift_descends_tendsto_metric[M: MetricSpace, N: MetricSpace](
    f: M -> N, p: Nat -> M, q: Nat -> N, b: N
) {
    is_sequence_lift(f, p, q) and tendsto_metric(compose(f, p), b) implies tendsto_metric(q, b)
} by {
    if is_sequence_lift(f, p, q) and tendsto_metric(compose(f, p), b) {
        let conv_pred: (Nat -> N) -> Bool = function(r: Nat -> N) {
            tendsto_metric(r, b)
        }
        conv_pred(compose(f, p))
        function_eq_transport_predicate(conv_pred, compose(f, p), q)
        conv_pred(q)
        tendsto_metric(q, b)
    }
}

/// Completeness transfers across a surjective isometry: every Cauchy sequence in
/// the target converges to the image of a source limit of a chosen right-inverse lift.
theorem surjective_isometry_cauchy_metric_imp_tendsto[M: CompleteMetricSpace, N: MetricSpace](
    f: M -> N, q: Nat -> N
) {
    is_isometry(f) and is_surjective_fn(f) and is_cauchy_metric(q) implies exists(b: N) {
        tendsto_metric(q, b)
    }
} by {
    if is_isometry(f) and is_surjective_fn(f) and is_cauchy_metric(q) {
        surjective_has_right_inverse(f)
        let g: N -> M satisfy {
            is_right_inverse_fn(f, g)
        }
        right_inverse_preserves_cauchy_metric(f, g, q)
        M.cauchy_metric_imp_tendsto(compose(g, q))
        let a: M satisfy {
            tendsto_metric(compose(g, q), a)
        }
        isometry_preserves_tendsto_metric(f, compose(g, q), a)
        right_inverse_compose_sequence(f, g, q)
        let conv_pred: (Nat -> N) -> Bool = function(r: Nat -> N) {
            tendsto_metric(r, f(a))
        }
        function_eq_transport_predicate(conv_pred, compose(f, compose(g, q)), q)
        conv_pred(q)
        exists(b: N) {
            tendsto_metric(q, b)
        }
    }
}
