/// Continuity in metric spaces.
///
/// The distance function of a metric space is jointly continuous in both
/// arguments, in the epsilon-delta sense, and continuous in each argument
/// separately.  The file also develops sequential continuity: a map is
/// sequentially continuous when it sends convergent sequences to convergent
/// sequences, every Lipschitz map is sequentially continuous, and every
/// continuous map between compatible metric topologies is sequentially
/// continuous.

from nat import Nat
from data.basic.set import Set, subset_contains, set_preimage, set_preimage_contains_eq
from data.basic.functions import compose
from analysis.metric.metric_space import MetricSpace, open_ball, open_ball_contains_iff,
    is_open_metric, is_lipschitz_with, lipschitz_with_preserves_tendsto_metric,
    tendsto_metric, metric_tail_bound, metric_tail_bound_at, metric_tail_bound_from_distances,
    tendsto_metric_tail_bound, tendsto_metric_from_tail_bounds
from analysis.metric.metric_topology import MetricTopologicalSpace,
    topological_open_is_metric_open, open_ball_is_topological_open
from analysis.topology.topological_space import is_continuous
from analysis.topology.continuous_preimage import continuous_open_preimage
from real import Real, eps_lt_half, bounds_imp_close, close_and_lt_imp_close,
    lt_add_right, add_assoc, add_comm, sub_cancels
from order import lt_trans, lte_lt_trans, lte_trans
from data.basic.logic import by_contradiction, not_forall_imp_exists_not, not_implies
numerals Real

/// Subtracting a summand from both sides of a strict inequality.
theorem sub_lt_of_lt_add(a: Real, b: Real, c: Real) {
    a < b + c implies a - c < b
} by {
    if a < b + c {
        lt_add_right(a, b + c, -c)
        a + -c < (b + c) + -c
        sub_cancels(b, c)
        b + c - c = b
        (b + c) + -c = b
        a + -c < b
        a - c < b
    }
}

/// The distance between `x2` and `y2` is at most the distance between `x` and `y`
/// plus the distances from `x2` to `x` and from `y2` to `y`.
theorem distance_difference_upper[M: MetricSpace](x: M, y: M, x2: M, y2: M) {
    x2.distance(y2) <= x.distance(y) + (x2.distance(x) + y2.distance(y))
} by {
    x2.distance(y2) <= x2.distance(x) + x.distance(y2)
    x.distance(y2) <= x.distance(y) + y.distance(y2)
    y.distance(y2) = y2.distance(y)
    x.distance(y2) <= x.distance(y) + y2.distance(y)
    x2.distance(x) + x.distance(y2) <= x2.distance(x) + (x.distance(y) + y2.distance(y))
    lte_trans(x2.distance(y2), x2.distance(x) + x.distance(y2),
        x2.distance(x) + (x.distance(y) + y2.distance(y)))
    add_assoc(x2.distance(x), x.distance(y), y2.distance(y))
    (x2.distance(x) + x.distance(y)) + y2.distance(y) =
        x2.distance(x) + (x.distance(y) + y2.distance(y))
    add_comm(x2.distance(x), x.distance(y))
    x2.distance(x) + x.distance(y) = x.distance(y) + x2.distance(x)
    add_assoc(x.distance(y), x2.distance(x), y2.distance(y))
    (x.distance(y) + x2.distance(x)) + y2.distance(y) =
        x.distance(y) + (x2.distance(x) + y2.distance(y))
    x2.distance(x) + (x.distance(y) + y2.distance(y)) =
        x.distance(y) + (x2.distance(x) + y2.distance(y))
    x2.distance(y2) <= x.distance(y) + (x2.distance(x) + y2.distance(y))
}

/// The distance between `x` and `y` is at most the distance between `x2` and `y2`
/// plus the distances from `x2` to `x` and from `y2` to `y`.
theorem distance_difference_lower[M: MetricSpace](x: M, y: M, x2: M, y2: M) {
    x.distance(y) <= x2.distance(y2) + (x2.distance(x) + y2.distance(y))
} by {
    x.distance(y) <= x.distance(x2) + x2.distance(y)
    x2.distance(y) <= x2.distance(y2) + y2.distance(y)
    x.distance(x2) = x2.distance(x)
    x.distance(x2) + x2.distance(y) <= x.distance(x2) + (x2.distance(y2) + y2.distance(y))
    lte_trans(x.distance(y), x.distance(x2) + x2.distance(y),
        x.distance(x2) + (x2.distance(y2) + y2.distance(y)))
    add_assoc(x.distance(x2), x2.distance(y2), y2.distance(y))
    (x.distance(x2) + x2.distance(y2)) + y2.distance(y) =
        x.distance(x2) + (x2.distance(y2) + y2.distance(y))
    add_comm(x.distance(x2), x2.distance(y2))
    x.distance(x2) + x2.distance(y2) = x2.distance(y2) + x.distance(x2)
    add_assoc(x2.distance(y2), x2.distance(x), y2.distance(y))
    (x2.distance(y2) + x2.distance(x)) + y2.distance(y) =
        x2.distance(y2) + (x2.distance(x) + y2.distance(y))
    x.distance(x2) + (x2.distance(y2) + y2.distance(y)) =
        x2.distance(y2) + (x2.distance(x) + y2.distance(y))
    x.distance(y) <= x2.distance(y2) + (x2.distance(x) + y2.distance(y))
}

/// Points within `delta` of `x` and within `delta` of `y` have pairwise distances
/// within `eps`, provided `delta + delta < eps`.
theorem distance_jointly_continuous_delta[M: MetricSpace](x: M, y: M, eps: Real, delta: Real) {
    delta.is_positive and delta + delta < eps implies
        forall(x2: M, y2: M) {
            (x2.distance(x) < delta and y2.distance(y) < delta) implies (x2.distance(y2)).is_close(x.distance(y), eps)
        }
} by {
    if delta.is_positive and delta + delta < eps {
        forall(x2: M, y2: M) {
            if (x2.distance(x) < delta and y2.distance(y) < delta) {
                x2.distance(x) < delta
                y2.distance(y) < delta
                x2.distance(x) + y2.distance(y) < delta + delta
                lt_trans(x2.distance(x) + y2.distance(y), delta + delta, eps)
                x2.distance(x) + y2.distance(y) < eps
                distance_difference_upper(x, y, x2, y2)
                x2.distance(y2) <= x.distance(y) + (x2.distance(x) + y2.distance(y))
                lt_add_right(x2.distance(x) + y2.distance(y), eps, x.distance(y))
                x.distance(y) + (x2.distance(x) + y2.distance(y)) < x.distance(y) + eps
                lte_lt_trans(x2.distance(y2),
                    x.distance(y) + (x2.distance(x) + y2.distance(y)),
                    x.distance(y) + eps)
                x2.distance(y2) < x.distance(y) + eps
                distance_difference_lower(x, y, x2, y2)
                x.distance(y) <= x2.distance(y2) + (x2.distance(x) + y2.distance(y))
                lt_add_right(x2.distance(x) + y2.distance(y), eps, x2.distance(y2))
                x2.distance(y2) + (x2.distance(x) + y2.distance(y)) < x2.distance(y2) + eps
                lte_lt_trans(x.distance(y),
                    x2.distance(y2) + (x2.distance(x) + y2.distance(y)),
                    x2.distance(y2) + eps)
                x.distance(y) < x2.distance(y2) + eps
                sub_lt_of_lt_add(x.distance(y), x2.distance(y2), eps)
                x.distance(y) - eps < x2.distance(y2)
                bounds_imp_close(x2.distance(y2), x.distance(y), eps)
                (x2.distance(y2)).is_close(x.distance(y), eps)
            }
        }
    }
}

/// The distance function is jointly continuous in both arguments: points close
/// to `x` and to `y` have pairwise distances close to `x.distance(y)`.
theorem metric_distance_jointly_continuous[M: MetricSpace] {
    forall(x: M, y: M, eps: Real) {
        eps.is_positive implies exists(delta: Real) {
            delta.is_positive and forall(x2: M, y2: M) {
                (x2.distance(x) < delta and y2.distance(y) < delta) implies (x2.distance(y2)).is_close(x.distance(y), eps)
            }
        }
    }
} by {
    if not forall(x: M, y: M, eps: Real) {
        eps.is_positive implies exists(delta: Real) {
            delta.is_positive and forall(x2: M, y2: M) {
                (x2.distance(x) < delta and y2.distance(y) < delta) implies (x2.distance(y2)).is_close(x.distance(y), eps)
            }
        }
    } {
        not_forall_imp_exists_not[M](function(x: M) {
            forall(y: M, eps: Real) {
                eps.is_positive implies exists(delta: Real) {
                    delta.is_positive and forall(x2: M, y2: M) {
                        (x2.distance(x) < delta and y2.distance(y) < delta) implies (x2.distance(y2)).is_close(x.distance(y), eps)
                    }
                }
            }
        })
        let x: M satisfy {
            not forall(y: M, eps: Real) {
                eps.is_positive implies exists(delta: Real) {
                    delta.is_positive and forall(x2: M, y2: M) {
                        (x2.distance(x) < delta and y2.distance(y) < delta) implies (x2.distance(y2)).is_close(x.distance(y), eps)
                    }
                }
            }
        }
        not_forall_imp_exists_not[M](function(y: M) {
            forall(eps: Real) {
                eps.is_positive implies exists(delta: Real) {
                    delta.is_positive and forall(x2: M, y2: M) {
                        (x2.distance(x) < delta and y2.distance(y) < delta) implies (x2.distance(y2)).is_close(x.distance(y), eps)
                    }
                }
            }
        })
        let y: M satisfy {
            not forall(eps: Real) {
                eps.is_positive implies exists(delta: Real) {
                    delta.is_positive and forall(x2: M, y2: M) {
                        (x2.distance(x) < delta and y2.distance(y) < delta) implies (x2.distance(y2)).is_close(x.distance(y), eps)
                    }
                }
            }
        }
        not_forall_imp_exists_not[Real](function(eps: Real) {
            eps.is_positive implies exists(delta: Real) {
                delta.is_positive and forall(x2: M, y2: M) {
                    (x2.distance(x) < delta and y2.distance(y) < delta) implies (x2.distance(y2)).is_close(x.distance(y), eps)
                }
            }
        })
        let eps: Real satisfy {
            not (eps.is_positive implies exists(delta: Real) {
                delta.is_positive and forall(x2: M, y2: M) {
                    (x2.distance(x) < delta and y2.distance(y) < delta) implies (x2.distance(y2)).is_close(x.distance(y), eps)
                }
            })
        }
        not_implies(eps.is_positive, exists(delta: Real) {
            delta.is_positive and forall(x2: M, y2: M) {
                (x2.distance(x) < delta and y2.distance(y) < delta) implies (x2.distance(y2)).is_close(x.distance(y), eps)
            }
        })
        (not (eps.is_positive implies exists(delta: Real) {
            delta.is_positive and forall(x2: M, y2: M) {
                (x2.distance(x) < delta and y2.distance(y) < delta) implies (x2.distance(y2)).is_close(x.distance(y), eps)
            }
        })) = (eps.is_positive and not exists(delta: Real) {
            delta.is_positive and forall(x2: M, y2: M) {
                (x2.distance(x) < delta and y2.distance(y) < delta) implies (x2.distance(y2)).is_close(x.distance(y), eps)
            }
        })
        eps.is_positive and not exists(delta: Real) {
            delta.is_positive and forall(x2: M, y2: M) {
                (x2.distance(x) < delta and y2.distance(y) < delta) implies (x2.distance(y2)).is_close(x.distance(y), eps)
            }
        }
        eps.is_positive
        eps_lt_half(eps)
        let delta: Real satisfy {
            delta.is_positive and delta + delta < eps
        }
        distance_jointly_continuous_delta(x, y, eps, delta)
        forall(x2: M, y2: M) {
            (x2.distance(x) < delta and y2.distance(y) < delta) implies (x2.distance(y2)).is_close(x.distance(y), eps)
        }
        delta.is_positive and forall(x2: M, y2: M) {
            (x2.distance(x) < delta and y2.distance(y) < delta) implies (x2.distance(y2)).is_close(x.distance(y), eps)
        }
        exists(d2: Real) {
            d2.is_positive and forall(x2: M, y2: M) {
                (x2.distance(x) < d2 and y2.distance(y) < d2) implies (x2.distance(y2)).is_close(x.distance(y), eps)
            }
        }
        not exists(d3: Real) {
            d3.is_positive and forall(x2: M, y2: M) {
                (x2.distance(x) < d3 and y2.distance(y) < d3) implies (x2.distance(y2)).is_close(x.distance(y), eps)
            }
        }
        false
    }
    by_contradiction(forall(x: M, y: M, eps: Real) {
        eps.is_positive implies exists(delta: Real) {
            delta.is_positive and forall(x2: M, y2: M) {
                (x2.distance(x) < delta and y2.distance(y) < delta) implies (x2.distance(y2)).is_close(x.distance(y), eps)
            }
        }
    })
    (not forall(x: M, y: M, eps: Real) {
        eps.is_positive implies exists(delta: Real) {
            delta.is_positive and forall(x2: M, y2: M) {
                (x2.distance(x) < delta and y2.distance(y) < delta) implies (x2.distance(y2)).is_close(x.distance(y), eps)
            }
        }
    } implies false) implies forall(x: M, y: M, eps: Real) {
        eps.is_positive implies exists(delta: Real) {
            delta.is_positive and forall(x2: M, y2: M) {
                (x2.distance(x) < delta and y2.distance(y) < delta) implies (x2.distance(y2)).is_close(x.distance(y), eps)
            }
        }
    }
    forall(x: M, y: M, eps: Real) {
        eps.is_positive implies exists(delta: Real) {
            delta.is_positive and forall(x2: M, y2: M) {
                (x2.distance(x) < delta and y2.distance(y) < delta) implies (x2.distance(y2)).is_close(x.distance(y), eps)
            }
        }
    }
}

/// Fixing the second argument, the distance map `x ↦ x.distance(y)` is continuous.
theorem metric_distance_first_continuous[M: MetricSpace] {
    forall(y: M, x: M, eps: Real) {
        eps.is_positive implies exists(delta: Real) {
            delta.is_positive and forall(x2: M) {
                (x2.distance(x) < delta) implies (x2.distance(y)).is_close(x.distance(y), eps)
            }
        }
    }
} by {
    if not forall(y: M, x: M, eps: Real) {
        eps.is_positive implies exists(delta: Real) {
            delta.is_positive and forall(x2: M) {
                (x2.distance(x) < delta) implies (x2.distance(y)).is_close(x.distance(y), eps)
            }
        }
    } {
        not_forall_imp_exists_not[M](function(y: M) {
            forall(x: M, eps: Real) {
                eps.is_positive implies exists(delta: Real) {
                    delta.is_positive and forall(x2: M) {
                        (x2.distance(x) < delta) implies (x2.distance(y)).is_close(x.distance(y), eps)
                    }
                }
            }
        })
        let y: M satisfy {
            not forall(x: M, eps: Real) {
                eps.is_positive implies exists(delta: Real) {
                    delta.is_positive and forall(x2: M) {
                        (x2.distance(x) < delta) implies (x2.distance(y)).is_close(x.distance(y), eps)
                    }
                }
            }
        }
        not_forall_imp_exists_not[M](function(x: M) {
            forall(eps: Real) {
                eps.is_positive implies exists(delta: Real) {
                    delta.is_positive and forall(x2: M) {
                        (x2.distance(x) < delta) implies (x2.distance(y)).is_close(x.distance(y), eps)
                    }
                }
            }
        })
        let x: M satisfy {
            not forall(eps: Real) {
                eps.is_positive implies exists(delta: Real) {
                    delta.is_positive and forall(x2: M) {
                        (x2.distance(x) < delta) implies (x2.distance(y)).is_close(x.distance(y), eps)
                    }
                }
            }
        }
        not_forall_imp_exists_not[Real](function(eps: Real) {
            eps.is_positive implies exists(delta: Real) {
                delta.is_positive and forall(x2: M) {
                    (x2.distance(x) < delta) implies (x2.distance(y)).is_close(x.distance(y), eps)
                }
            }
        })
        let eps: Real satisfy {
            not (eps.is_positive implies exists(delta: Real) {
                delta.is_positive and forall(x2: M) {
                    (x2.distance(x) < delta) implies (x2.distance(y)).is_close(x.distance(y), eps)
                }
            })
        }
        not_implies(eps.is_positive, exists(delta: Real) {
            delta.is_positive and forall(x2: M) {
                (x2.distance(x) < delta) implies (x2.distance(y)).is_close(x.distance(y), eps)
            }
        })
        (not (eps.is_positive implies exists(delta: Real) {
            delta.is_positive and forall(x2: M) {
                (x2.distance(x) < delta) implies (x2.distance(y)).is_close(x.distance(y), eps)
            }
        })) = (eps.is_positive and not exists(delta: Real) {
            delta.is_positive and forall(x2: M) {
                (x2.distance(x) < delta) implies (x2.distance(y)).is_close(x.distance(y), eps)
            }
        })
        eps.is_positive and not exists(delta: Real) {
            delta.is_positive and forall(x2: M) {
                (x2.distance(x) < delta) implies (x2.distance(y)).is_close(x.distance(y), eps)
            }
        }
        eps.is_positive
        metric_distance_jointly_continuous[M]
        forall(x3: M, y3: M, e3: Real) {
            e3.is_positive implies exists(delta: Real) {
                delta.is_positive and forall(x2: M, y2: M) {
                    (x2.distance(x3) < delta and y2.distance(y3) < delta) implies (x2.distance(y2)).is_close(x3.distance(y3), e3)
                }
            }
        }
        let delta: Real satisfy {
            delta.is_positive and forall(x2: M, y2: M) {
                (x2.distance(x) < delta and y2.distance(y) < delta) implies (x2.distance(y2)).is_close(x.distance(y), eps)
            }
        }
        delta.is_positive
        forall(x2: M) {
            if x2.distance(x) < delta {
                y.distance(y) = 0
                Real.0 < delta
                y.distance(y) < delta
                x2.distance(x) < delta and y.distance(y) < delta
                (x2.distance(y)).is_close(x.distance(y), eps)
            }
        }
        delta.is_positive and forall(x2: M) {
            (x2.distance(x) < delta) implies (x2.distance(y)).is_close(x.distance(y), eps)
        }
        exists(d2: Real) {
            d2.is_positive and forall(x2: M) {
                (x2.distance(x) < d2) implies (x2.distance(y)).is_close(x.distance(y), eps)
            }
        }
        not exists(d3: Real) {
            d3.is_positive and forall(x2: M) {
                (x2.distance(x) < d3) implies (x2.distance(y)).is_close(x.distance(y), eps)
            }
        }
        false
    }
    by_contradiction(forall(y: M, x: M, eps: Real) {
        eps.is_positive implies exists(delta: Real) {
            delta.is_positive and forall(x2: M) {
                (x2.distance(x) < delta) implies (x2.distance(y)).is_close(x.distance(y), eps)
            }
        }
    })
    (not forall(y: M, x: M, eps: Real) {
        eps.is_positive implies exists(delta: Real) {
            delta.is_positive and forall(x2: M) {
                (x2.distance(x) < delta) implies (x2.distance(y)).is_close(x.distance(y), eps)
            }
        }
    } implies false) implies forall(y: M, x: M, eps: Real) {
        eps.is_positive implies exists(delta: Real) {
            delta.is_positive and forall(x2: M) {
                (x2.distance(x) < delta) implies (x2.distance(y)).is_close(x.distance(y), eps)
            }
        }
    }
    forall(y: M, x: M, eps: Real) {
        eps.is_positive implies exists(delta: Real) {
            delta.is_positive and forall(x2: M) {
                (x2.distance(x) < delta) implies (x2.distance(y)).is_close(x.distance(y), eps)
            }
        }
    }
}

/// True if `f` sends convergent sequences to convergent sequences.
define is_sequentially_continuous[M: MetricSpace, N: MetricSpace](f: M -> N) -> Bool {
    forall(q: Nat -> M, a: M) {
        tendsto_metric(q, a) implies tendsto_metric(compose(f, q), f(a))
    }
}

/// Unfolding lemma for `is_sequentially_continuous`.
theorem is_sequentially_continuous_iff[M: MetricSpace, N: MetricSpace](f: M -> N) {
    is_sequentially_continuous(f) = forall(q: Nat -> M, a: M) {
        tendsto_metric(q, a) implies tendsto_metric(compose(f, q), f(a))
    }
} by {
}

/// A Lipschitz map preserves convergence of sequences.
theorem lipschitz_with_is_sequentially_continuous[M: MetricSpace, N: MetricSpace](
    k: Real, f: M -> N) {
    is_lipschitz_with(k, f) implies is_sequentially_continuous(f)
} by {
    if is_lipschitz_with(k, f) {
        is_sequentially_continuous_iff(f)
        forall(q: Nat -> M, a: M) {
            if tendsto_metric(q, a) {
                lipschitz_with_preserves_tendsto_metric(k, f, q, a)
                tendsto_metric(compose(f, q), f(a))
            }
        }
        is_sequentially_continuous(f)
    }
}

/// A continuous map between compatible metric topologies preserves convergence
/// of sequences.
theorem continuous_imp_sequentially_continuous[M: MetricTopologicalSpace, N: MetricTopologicalSpace](
    f: M -> N) {
    is_continuous(f) implies is_sequentially_continuous(f)
} by {
    if is_continuous(f) {
        is_sequentially_continuous_iff(f)
        forall(q: Nat -> M, a: M) {
            if tendsto_metric(q, a) {
                forall(eps: Real) {
                    if eps.is_positive {
                        open_ball_is_topological_open[N](f(a), eps)
                        N.is_open(open_ball(f(a), eps))
                        continuous_open_preimage[M, N](f, open_ball(f(a), eps))
                        M.is_open(set_preimage(f, open_ball(f(a), eps)))
                        open_ball_contains_iff(f(a), eps, f(a))
                        f(a).distance(f(a)) < eps
                        set_preimage_contains_eq(f, open_ball(f(a), eps), a)
                        set_preimage(f, open_ball(f(a), eps)).contains(a)
                        topological_open_is_metric_open(set_preimage(f, open_ball(f(a), eps)))
                        is_open_metric(set_preimage(f, open_ball(f(a), eps)))
                        is_open_metric(set_preimage(f, open_ball(f(a), eps))) = forall(z: M) {
                            set_preimage(f, open_ball(f(a), eps)).contains(z) implies exists(r: Real) {
                                r.is_positive and open_ball(z, r).subset(set_preimage(f, open_ball(f(a), eps)))
                            }
                        }
                        let r: Real satisfy {
                            r.is_positive and open_ball(a, r).subset(set_preimage(f, open_ball(f(a), eps)))
                        }
                        tendsto_metric_tail_bound(q, a, r)
                        let n: Nat satisfy {
                            metric_tail_bound(q, a, n, r)
                        }
                        forall(i: Nat) {
                            if n <= i {
                                metric_tail_bound_at(q, a, n, r, i)
                                q(i).distance(a) < r
                                open_ball(a, r).contains(q(i))
                                subset_contains(open_ball(a, r),
                                    set_preimage(f, open_ball(f(a), eps)), q(i))
                                set_preimage(f, open_ball(f(a), eps)).contains(q(i))
                                set_preimage_contains_eq(f, open_ball(f(a), eps), q(i))
                                open_ball(f(a), eps).contains(f(q(i)))
                                open_ball_contains_iff(f(a), eps, f(q(i)))
                                f(q(i)).distance(f(a)) < eps
                                compose(f, q)(i).distance(f(a)) < eps
                            }
                        }
                        metric_tail_bound(compose(f, q), f(a), n, eps)
                        exists(m: Nat) { metric_tail_bound(compose(f, q), f(a), m, eps) }
                    }
                }
                if not tendsto_metric(compose(f, q), f(a)) {
                    tendsto_metric(compose(f, q), f(a)) = forall(e: Real) {
                        e.is_positive implies exists(n: Nat) {
                            metric_tail_bound(compose(f, q), f(a), n, e)
                        }
                    }
                    not forall(e: Real) {
                        e.is_positive implies exists(n: Nat) {
                            metric_tail_bound(compose(f, q), f(a), n, e)
                        }
                    }
                    not_forall_imp_exists_not[Real](function(e: Real) {
                        e.is_positive implies exists(n: Nat) {
                            metric_tail_bound(compose(f, q), f(a), n, e)
                        }
                    })
                    let e: Real satisfy {
                        not (e.is_positive implies exists(n: Nat) {
                            metric_tail_bound(compose(f, q), f(a), n, e)
                        })
                    }
                    not_implies(e.is_positive, exists(n: Nat) {
                        metric_tail_bound(compose(f, q), f(a), n, e)
                    })
                    (not (e.is_positive implies exists(n: Nat) {
                        metric_tail_bound(compose(f, q), f(a), n, e)
                    })) = (e.is_positive and not exists(n: Nat) {
                        metric_tail_bound(compose(f, q), f(a), n, e)
                    })
                    e.is_positive and not exists(n: Nat) {
                        metric_tail_bound(compose(f, q), f(a), n, e)
                    }
                    e.is_positive
                    forall(e2: Real) {
                        e2.is_positive implies exists(m: Nat) {
                            metric_tail_bound(compose(f, q), f(a), m, e2)
                        }
                    }
                    e.is_positive implies exists(m: Nat) {
                        metric_tail_bound(compose(f, q), f(a), m, e)
                    }
                    exists(m: Nat) {
                        metric_tail_bound(compose(f, q), f(a), m, e)
                    }
                    not exists(n: Nat) {
                        metric_tail_bound(compose(f, q), f(a), n, e)
                    }
                    false
                }
                by_contradiction(tendsto_metric(compose(f, q), f(a)))
                (not tendsto_metric(compose(f, q), f(a)) implies false) implies tendsto_metric(compose(f, q), f(a))
                tendsto_metric(compose(f, q), f(a))
            }
        }
        is_sequentially_continuous(f)
    }
}
