/// Sequential compactness in metric spaces.
///
/// A set is sequentially compact when every sequence in it has a subsequence
/// converging to a point of the set.  Sequentially compact metric spaces are
/// complete (a Cauchy sequence with a convergent subsequence converges), and
/// closed intervals of the real line are sequentially compact via the
/// Bolzano-Weierstrass theorem.  Together with the Heine-Borel theorem this
/// makes sequential compactness and open-cover compactness coincide for
/// closed intervals.  Finally, continuous images of sequentially compact
/// sets are sequentially compact.

from nat import Nat, from_nat, lte_add_left, lte_add_right
from order import is_monotone, lt_imp_lte, lte_trans, monotone_apply,
    closed_interval, lte_lt_trans, lt_lte_trans, lt_trans, not_lte_imp_gt, not_lt_self
from real import Real, is_unbounded, converges_to, subsequence, bolzano_weierstrass,
    from_nat_lte_mono, lte_abs, add_comm, add_assoc, abs_neg, add_neg_eq_zero,
    add_zero_right, add_zero_left, neg_distrib, neg_neg
from data.basic.set import Set, set_image, set_image_contains_eq,
    set_image_contains_witness, universal_set_contains_eq, image_contains, subset_contains
from data.basic.functions import compose, function_extensionality, compose_assoc
from data.basic.logic import by_contradiction, not_forall_imp_exists_not, not_implies
from analysis.metric.metric_space import MetricSpace, tendsto_metric, metric_tail_bound,
    metric_tail_bound_at, metric_tail_bound_from_distances, tendsto_metric_tail_bound,
    tendsto_metric_from_tail_bounds, is_cauchy_metric, cauchy_metric_bound,
    cauchy_metric_bound_at, is_cauchy_metric_bound, tendsto_metric_unique, is_bounded,
    closed_ball, closed_ball_contains_iff
from analysis.metric.metric_sequence import nat_zero_lte, metric_subsequence_tendsto,
    tendsto_metric_in_closed_interval, sequence_range, sequence_range_contains_eq,
    tendsto_metric_seq_is_bounded
from analysis.metric.complete_metric_space import real_distance
// Importing this module brings the `Real: MetricTopologicalSpace` instance into scope.
from analysis.metric.real_metric_topology import closed_interval_is_closed_metric
from analysis.metric.metric_continuity import is_sequentially_continuous,
    is_sequentially_continuous_iff, continuous_imp_sequentially_continuous
from analysis.metric.metric_topology import MetricTopologicalSpace
from analysis.metric.complete_metric_space import real_converges_to_imp_tendsto_metric
from analysis.metric.metric_separation_compact import metric_space_point_exists
from analysis.real.real_archimedean import real_lt_some_nat
from algebra.add_ordered_group import add_le_add_right
from analysis.topology.topological_space import is_compact, is_continuous
from analysis.topology.real_topology import heine_borel_closed_interval
from order_set import closed_interval_set, closed_interval_set_contains_eq

numerals Real

/// True if every sequence in `s` has a subsequence converging to a point of
/// `s`, with the subsequence encoded by a nondecreasing unbounded index map.
define is_sequentially_compact[M: MetricSpace](s: Set[M]) -> Bool {
    forall(q: Nat -> M) {
        (forall(n: Nat) { s.contains(q(n)) }) implies exists(f: Nat -> Nat, a: M) {
            is_monotone(f) and is_unbounded(f) and s.contains(a) and tendsto_metric(compose(q, f), a)
        }
    }
}

/// Unfolding lemma for `is_sequentially_compact`.
theorem is_sequentially_compact_iff[M: MetricSpace](s: Set[M]) {
    is_sequentially_compact(s) = forall(q: Nat -> M) {
        (forall(n: Nat) { s.contains(q(n)) }) implies exists(f: Nat -> Nat, a: M) {
            is_monotone(f) and is_unbounded(f) and s.contains(a) and tendsto_metric(compose(q, f), a)
        }
    }
} by {
}

/// A sequentially compact set has a convergent subsequence for every
/// sequence in it.
theorem sequentially_compact_subsequence[M: MetricSpace](s: Set[M], p: Nat -> M) {
    is_sequentially_compact(s) and (forall(n: Nat) { s.contains(p(n)) }) implies exists(f: Nat -> Nat, b: M) {
        is_monotone(f) and is_unbounded(f) and s.contains(b) and tendsto_metric(compose(p, f), b)
    }
} by {
    if is_sequentially_compact(s) and (forall(n: Nat) { s.contains(p(n)) }) {
        is_sequentially_compact_iff(s)
        is_sequentially_compact(s) = forall(q0: Nat -> M) {
            (forall(n: Nat) { s.contains(q0(n)) }) implies exists(f: Nat -> Nat, b: M) {
                is_monotone(f) and is_unbounded(f) and s.contains(b) and tendsto_metric(compose(q0, f), b)
            }
        }
        exists(f: Nat -> Nat, b: M) {
            is_monotone(f) and is_unbounded(f) and s.contains(b) and tendsto_metric(compose(p, f), b)
        }
    }
}

/// A Cauchy sequence with a convergent subsequence converges to the
/// subsequence limit.
///
/// For a given `eps`, take `half` with `half + half < eps`, a Cauchy threshold
/// `n1` and a subsequence tail threshold `n2`.  Unboundedness of the index
/// map provides an index `k1` whose image lies beyond `n1`; for every index
/// `i` at least `n1 + k1 + n2`, both `q(i)` and `q(f(i))` are within `half`
/// of each other and `q(f(i))` is within `half` of the limit.
theorem cauchy_metric_convergent_subsequence_imp_convergent[M: MetricSpace](
    q: Nat -> M, f: Nat -> Nat, a: M) {
    is_cauchy_metric(q) and is_monotone(f) and is_unbounded(f) and tendsto_metric(compose(q, f), a) implies tendsto_metric(q, a)
} by {
    if is_cauchy_metric(q) and is_monotone(f) and is_unbounded(f) and tendsto_metric(compose(q, f), a) {
        forall(eps: Real) {
            if eps.is_positive {
                let half: Real satisfy {
                    half.is_positive and half + half < eps
                }
                is_cauchy_metric_bound(q, half)
                let n1: Nat satisfy {
                    cauchy_metric_bound(q, n1, half)
                }
                tendsto_metric_tail_bound(compose(q, f), a, half)
                let n2: Nat satisfy {
                    metric_tail_bound(compose(q, f), a, n2, half)
                }
                is_unbounded(f) = forall(bound: Nat) {
                    exists(k: Nat) { bound < f(k) }
                }
                exists(k: Nat) { n1 < f(k) }
                let k1: Nat satisfy {
                    n1 < f(k1)
                }
                let m: Nat = n1 + k1 + n2
                lte_add_left(n1, Nat.0, k1)
                nat_zero_lte(k1)
                n1 + Nat.0 <= n1 + k1
                n1 + Nat.0 = n1
                n1 <= n1 + k1
                lte_add_left(n1 + k1, Nat.0, n2)
                nat_zero_lte(n2)
                (n1 + k1) + Nat.0 <= (n1 + k1) + n2
                (n1 + k1) + Nat.0 = n1 + k1
                n1 + k1 <= m
                lte_trans(n1, n1 + k1, m)
                n1 <= m
                lte_add_right(k1, Nat.0, n1)
                nat_zero_lte(n1)
                Nat.0 + k1 <= n1 + k1
                Nat.0 + k1 = k1
                k1 <= n1 + k1
                lte_trans(k1, n1 + k1, m)
                k1 <= m
                lte_add_right(n2, Nat.0, n1 + k1)
                nat_zero_lte(n1 + k1)
                Nat.0 + n2 <= (n1 + k1) + n2
                Nat.0 + n2 = n2
                n2 <= m
                forall(i: Nat) {
                    if m <= i {
                        lte_trans(n1, m, i)
                        n1 <= i
                        lte_trans(k1, m, i)
                        k1 <= i
                        lte_trans(n2, m, i)
                        n2 <= i
                        monotone_apply(f, k1, i)
                        f(k1) <= f(i)
                        lt_imp_lte(n1, f(k1))
                        n1 <= f(k1)
                        lte_trans(n1, f(k1), f(i))
                        n1 <= f(i)
                        cauchy_metric_bound_at(q, n1, half, i, f(i))
                        q(i).distance(q(f(i))) < half
                        metric_tail_bound_at(compose(q, f), a, n2, half, i)
                        compose(q, f)(i).distance(a) < half
                        q(f(i)).distance(a) < half
                        q(i).distance(a) <= q(i).distance(q(f(i))) + q(f(i)).distance(a)
                        q(i).distance(q(f(i))) + q(f(i)).distance(a) < half + half
                        lte_lt_trans(q(i).distance(a),
                            q(i).distance(q(f(i))) + q(f(i)).distance(a), half + half)
                        q(i).distance(a) < half + half
                        half + half < eps
                        lt_trans(q(i).distance(a), half + half, eps)
                        q(i).distance(a) < eps
                    }
                }
                metric_tail_bound_from_distances(q, a, m, eps)
                metric_tail_bound(q, a, m, eps)
                exists(m0: Nat) {
                    metric_tail_bound(q, a, m0, eps)
                }
            }
        }
        if not tendsto_metric(q, a) {
            tendsto_metric(q, a) = forall(e: Real) {
                e.is_positive implies exists(n: Nat) {
                    metric_tail_bound(q, a, n, e)
                }
            }
            not forall(e: Real) {
                e.is_positive implies exists(n: Nat) {
                    metric_tail_bound(q, a, n, e)
                }
            }
            not_forall_imp_exists_not[Real](function(e: Real) {
                e.is_positive implies exists(n: Nat) {
                    metric_tail_bound(q, a, n, e)
                }
            })
            let e: Real satisfy {
                not (e.is_positive implies exists(n: Nat) {
                    metric_tail_bound(q, a, n, e)
                })
            }
            not_implies(e.is_positive, exists(n: Nat) {
                metric_tail_bound(q, a, n, e)
            })
            (not (e.is_positive implies exists(n: Nat) {
                metric_tail_bound(q, a, n, e)
            })) = (e.is_positive and not exists(n: Nat) {
                metric_tail_bound(q, a, n, e)
            })
            e.is_positive and not exists(n: Nat) {
                metric_tail_bound(q, a, n, e)
            }
            e.is_positive
            forall(e2: Real) {
                e2.is_positive implies exists(m0: Nat) {
                    metric_tail_bound(q, a, m0, e2)
                }
            }
            e.is_positive implies exists(m0: Nat) {
                metric_tail_bound(q, a, m0, e)
            }
            exists(m0: Nat) {
                metric_tail_bound(q, a, m0, e)
            }
            not exists(n: Nat) {
                metric_tail_bound(q, a, n, e)
            }
            false
        }
        by_contradiction(tendsto_metric(q, a))
        tendsto_metric(q, a)
    }
}

/// A sequentially compact metric space is complete: every Cauchy sequence
/// converges.
theorem sequentially_compact_imp_complete[M: MetricSpace] {
    is_sequentially_compact[M](Set[M].universal_set) implies forall(q: Nat -> M) {
        is_cauchy_metric(q) implies exists(a: M) { tendsto_metric(q, a) }
    }
} by {
    if is_sequentially_compact[M](Set[M].universal_set) {
        forall(q: Nat -> M) {
            if is_cauchy_metric(q) {
                forall(n: Nat) {
                    (Set[M].universal_set).contains(q(n))
                }
                sequentially_compact_subsequence[M](Set[M].universal_set, q)
                exists(f: Nat -> Nat, a: M) {
                    is_monotone(f) and is_unbounded(f) and (Set[M].universal_set).contains(a) and tendsto_metric(compose(q, f), a)
                }
                let f: Nat -> Nat satisfy {
                    exists(a: M) {
                        is_monotone(f) and is_unbounded(f) and (Set[M].universal_set).contains(a) and tendsto_metric(compose(q, f), a)
                    }
                }
                let a: M satisfy {
                    is_monotone(f) and is_unbounded(f) and (Set[M].universal_set).contains(a) and tendsto_metric(compose(q, f), a)
                }
                is_monotone(f)
                is_unbounded(f)
                tendsto_metric(compose(q, f), a)
                cauchy_metric_convergent_subsequence_imp_convergent(q, f, a)
                tendsto_metric(q, a)
                exists(b: M) { tendsto_metric(q, b) }
            }
        }
        forall(q: Nat -> M) {
            is_cauchy_metric(q) implies exists(a: M) { tendsto_metric(q, a) }
        }
    }
}

/// A sequentially compact set contains the limits of all its sequences.
///
/// A convergent sequence in `s` has a subsequence converging to a point of
/// `s`; the subsequence also converges to the limit of the original sequence,
/// and limits are unique.
theorem sequentially_compact_contains_limits[M: MetricSpace](s: Set[M], q: Nat -> M, a: M) {
    is_sequentially_compact(s) and (forall(n: Nat) { s.contains(q(n)) }) and tendsto_metric(q, a) implies s.contains(a)
} by {
    if is_sequentially_compact(s) and (forall(n: Nat) { s.contains(q(n)) }) and tendsto_metric(q, a) {
        is_sequentially_compact_iff(s)
        is_sequentially_compact(s) = forall(q0: Nat -> M) {
            (forall(n: Nat) { s.contains(q0(n)) }) implies exists(f: Nat -> Nat, b: M) {
                is_monotone(f) and is_unbounded(f) and s.contains(b) and tendsto_metric(compose(q0, f), b)
            }
        }
        exists(f: Nat -> Nat, b: M) {
            is_monotone(f) and is_unbounded(f) and s.contains(b) and tendsto_metric(compose(q, f), b)
        }
        let f: Nat -> Nat satisfy {
            exists(b: M) {
                is_monotone(f) and is_unbounded(f) and s.contains(b) and tendsto_metric(compose(q, f), b)
            }
        }
        let b: M satisfy {
            is_monotone(f) and is_unbounded(f) and s.contains(b) and tendsto_metric(compose(q, f), b)
        }
        is_monotone(f)
        is_unbounded(f)
        metric_subsequence_tendsto(q, f, a)
        tendsto_metric(compose(q, f), a)
        tendsto_metric_unique(compose(q, f), a, b)
        a = b
        s.contains(b)
        s.contains(a)
    }
}

/// A closed interval of the real line is sequentially compact.
///
/// By the Bolzano-Weierstrass theorem a sequence in `[lo, hi]` has a
/// convergent subsequence, and the limit of that subsequence lies in the
/// interval because the interval is closed under limits of sequences.
theorem closed_interval_is_sequentially_compact(lo: Real, hi: Real) {
    is_sequentially_compact[Real](closed_interval_set(lo, hi))
} by {
    is_sequentially_compact_iff[Real](closed_interval_set(lo, hi))
    is_sequentially_compact[Real](closed_interval_set(lo, hi)) = forall(q: Nat -> Real) {
        (forall(n: Nat) { closed_interval_set(lo, hi).contains(q(n)) }) implies exists(f: Nat -> Nat, l: Real) {
            is_monotone(f) and is_unbounded(f) and closed_interval_set(lo, hi).contains(l) and tendsto_metric(compose(q, f), l)
        }
    }
    forall(q: Nat -> Real) {
        if (forall(n: Nat) { closed_interval_set(lo, hi).contains(q(n)) }) {
            forall(n: Nat) {
                closed_interval_set_contains_eq(lo, hi, q(n))
                closed_interval(lo, hi, q(n)) = (lo <= q(n) and q(n) <= hi)
                lo <= q(n) and q(n) <= hi
            }
            exists(lo0: Real, hi0: Real) {
                forall(n: Nat) { lo0 <= q(n) and q(n) <= hi0 }
            }
            bolzano_weierstrass(q)
            exists(f: Nat -> Nat, l: Real) {
                is_monotone(f) and is_unbounded(f) and converges_to(subsequence(q, f), l)
            }
            let f: Nat -> Nat satisfy {
                exists(l: Real) {
                    is_monotone(f) and is_unbounded(f) and converges_to(subsequence(q, f), l)
                }
            }
            let l: Real satisfy {
                is_monotone(f) and is_unbounded(f) and converges_to(subsequence(q, f), l)
            }
            is_monotone(f)
            is_unbounded(f)
            real_converges_to_imp_tendsto_metric(subsequence(q, f), l)
            tendsto_metric(subsequence(q, f), l)
            forall(n: Nat) {
                subsequence(q, f, n) = q(f(n))
                closed_interval_set(lo, hi).contains(q(f(n)))
                closed_interval_set(lo, hi).contains(subsequence(q, f, n))
            }
            tendsto_metric_in_closed_interval(subsequence(q, f), l, lo, hi)
            closed_interval_set(lo, hi).contains(l)
            function_extensionality[Nat, Real](subsequence(q, f), compose(q, f))
            subsequence(q, f) = compose(q, f)
            tendsto_metric(compose(q, f), l)
            is_monotone(f) and is_unbounded(f) and closed_interval_set(lo, hi).contains(l) and tendsto_metric(compose(q, f), l)
            exists(g: Nat -> Nat, l2: Real) {
                is_monotone(g) and is_unbounded(g) and closed_interval_set(lo, hi).contains(l2) and tendsto_metric(compose(q, g), l2)
            }
        }
    }
    if not is_sequentially_compact[Real](closed_interval_set(lo, hi)) {
        is_sequentially_compact[Real](closed_interval_set(lo, hi)) = forall(q: Nat -> Real) {
            (forall(n: Nat) { closed_interval_set(lo, hi).contains(q(n)) }) implies exists(f: Nat -> Nat, l: Real) {
                is_monotone(f) and is_unbounded(f) and closed_interval_set(lo, hi).contains(l) and tendsto_metric(compose(q, f), l)
            }
        }
        not forall(q: Nat -> Real) {
            (forall(n: Nat) { closed_interval_set(lo, hi).contains(q(n)) }) implies exists(f: Nat -> Nat, l: Real) {
                is_monotone(f) and is_unbounded(f) and closed_interval_set(lo, hi).contains(l) and tendsto_metric(compose(q, f), l)
            }
        }
        not_forall_imp_exists_not[Nat -> Real](function(q: Nat -> Real) {
            (forall(n: Nat) { closed_interval_set(lo, hi).contains(q(n)) }) implies exists(f: Nat -> Nat, l: Real) {
                is_monotone(f) and is_unbounded(f) and closed_interval_set(lo, hi).contains(l) and tendsto_metric(compose(q, f), l)
            }
        })
        let q: Nat -> Real satisfy {
            not ((forall(n: Nat) { closed_interval_set(lo, hi).contains(q(n)) }) implies exists(f: Nat -> Nat, l: Real) {
                    is_monotone(f) and is_unbounded(f) and closed_interval_set(lo, hi).contains(l) and tendsto_metric(compose(q, f), l)
                })
        }
        not_implies((forall(n: Nat) { closed_interval_set(lo, hi).contains(q(n)) }),
            exists(f: Nat -> Nat, l: Real) {
                is_monotone(f) and is_unbounded(f) and closed_interval_set(lo, hi).contains(l) and tendsto_metric(compose(q, f), l)
            })
        (not ((forall(n: Nat) { closed_interval_set(lo, hi).contains(q(n)) }) implies exists(f: Nat -> Nat, l: Real) {
                is_monotone(f) and is_unbounded(f) and closed_interval_set(lo, hi).contains(l) and tendsto_metric(compose(q, f), l)
            })) = ((forall(n: Nat) { closed_interval_set(lo, hi).contains(q(n)) }) and not exists(f: Nat -> Nat, l: Real) {
                    is_monotone(f) and is_unbounded(f) and closed_interval_set(lo, hi).contains(l) and tendsto_metric(compose(q, f), l)
                })
        (forall(n: Nat) { closed_interval_set(lo, hi).contains(q(n)) }) and not exists(f: Nat -> Nat, l: Real) {
                is_monotone(f) and is_unbounded(f) and closed_interval_set(lo, hi).contains(l) and tendsto_metric(compose(q, f), l)
            }
        forall(n: Nat) { closed_interval_set(lo, hi).contains(q(n)) }
        (forall(n: Nat) { closed_interval_set(lo, hi).contains(q(n)) }) implies exists(f: Nat -> Nat, l: Real) {
            is_monotone(f) and is_unbounded(f) and closed_interval_set(lo, hi).contains(l) and tendsto_metric(compose(q, f), l)
        }
        exists(f: Nat -> Nat, l: Real) {
            is_monotone(f) and is_unbounded(f) and closed_interval_set(lo, hi).contains(l) and tendsto_metric(compose(q, f), l)
        }
        not exists(f: Nat -> Nat, l: Real) {
            is_monotone(f) and is_unbounded(f) and closed_interval_set(lo, hi).contains(l) and tendsto_metric(compose(q, f), l)
        }
        false
    }
    by_contradiction(is_sequentially_compact[Real](closed_interval_set(lo, hi)))
    is_sequentially_compact[Real](closed_interval_set(lo, hi))
}

/// For closed intervals, sequential compactness coincides with open-cover
/// compactness (Heine-Borel).
theorem closed_interval_seq_compact_iff_compact(lo: Real, hi: Real) {
    is_sequentially_compact[Real](closed_interval_set(lo, hi)) = is_compact[Real](closed_interval_set(lo, hi))
} by {
    if is_sequentially_compact[Real](closed_interval_set(lo, hi)) {
        heine_borel_closed_interval(lo, hi)
        is_compact[Real](closed_interval_set(lo, hi))
    }
    if is_compact[Real](closed_interval_set(lo, hi)) {
        closed_interval_is_sequentially_compact(lo, hi)
        is_sequentially_compact[Real](closed_interval_set(lo, hi))
    }
}

/// The continuous image of a sequentially compact set is sequentially
/// compact.
///
/// A sequence in the image lifts pointwise to a sequence in the source set;
/// a subsequence of the lift converges to a point of the source set, and
/// continuity sends that to a convergent subsequence of the image sequence
/// whose limit is the image of the source limit.
theorem sequentially_compact_image[M: MetricTopologicalSpace, N: MetricTopologicalSpace](
    s: Set[M], f: M -> N) {
    is_sequentially_compact(s) and is_continuous(f) implies is_sequentially_compact(set_image(s, f))
} by {
    if is_sequentially_compact(s) and is_continuous(f) {
        is_sequentially_compact_iff(set_image(s, f))
        forall(q: Nat -> N) {
            if (forall(n: Nat) { set_image(s, f).contains(q(n)) }) {
                forall(n: Nat) {
                    set_image_contains_witness(s, f, q(n))
                    exists(x: M) {
                        s.contains(x) and q(n) = f(x)
                    }
                }
                let p(n: Nat) -> result: M satisfy {
                    s.contains(result) and f(result) = q(n)
                }
                forall(n: Nat) {
                    s.contains(p(n))
                }
                sequentially_compact_subsequence(s, p)
                exists(g: Nat -> Nat, b: M) {
                    is_monotone(g) and is_unbounded(g) and s.contains(b) and tendsto_metric(compose(p, g), b)
                }
                let g: Nat -> Nat satisfy {
                    exists(b: M) {
                        is_monotone(g) and is_unbounded(g) and s.contains(b) and tendsto_metric(compose(p, g), b)
                    }
                }
                let b: M satisfy {
                    is_monotone(g) and is_unbounded(g) and s.contains(b) and tendsto_metric(compose(p, g), b)
                }
                is_monotone(g)
                is_unbounded(g)
                tendsto_metric(compose(p, g), b)
                continuous_imp_sequentially_continuous(f)
                is_sequentially_continuous(f) = forall(q1: Nat -> M, a1: M) {
                    tendsto_metric(q1, a1) implies tendsto_metric(compose(f, q1), f(a1))
                }
                tendsto_metric(compose(p, g), b) implies tendsto_metric(compose(f, compose(p, g)), f(b))
                tendsto_metric(compose(f, compose(p, g)), f(b))
                compose_assoc(f, p, g)
                compose(f, compose(p, g)) = compose(compose(f, p), g)
                tendsto_metric(compose(compose(f, p), g), f(b))
                forall(n: Nat) {
                    f(p(n)) = q(n)
                }
                function_extensionality[Nat, N](compose(f, p), q)
                compose(f, p) = q
                tendsto_metric(compose(q, g), f(b))
                set_image_contains_eq(s, f, f(b))
                set_image(s, f).contains(f(b)) = image_contains(f, s, f(b))
                image_contains(f, s, f(b)) = exists(x: M) {
                    s.contains(x) and f(b) = f(x)
                }
                s.contains(b) and f(b) = f(b)
                image_contains(f, s, f(b))
                set_image(s, f).contains(f(b))
                is_monotone(g) and is_unbounded(g) and set_image(s, f).contains(f(b)) and tendsto_metric(compose(q, g), f(b))
                exists(h: Nat -> Nat, c: N) {
                    is_monotone(h) and is_unbounded(h) and set_image(s, f).contains(c) and tendsto_metric(compose(q, h), c)
                }
            }
        }
        if not is_sequentially_compact(set_image(s, f)) {
            is_sequentially_compact(set_image(s, f)) = forall(q: Nat -> N) {
                (forall(n: Nat) { set_image(s, f).contains(q(n)) }) implies exists(h: Nat -> Nat, c: N) {
                    is_monotone(h) and is_unbounded(h) and set_image(s, f).contains(c) and tendsto_metric(compose(q, h), c)
                }
            }
            not forall(q: Nat -> N) {
                (forall(n: Nat) { set_image(s, f).contains(q(n)) }) implies exists(h: Nat -> Nat, c: N) {
                    is_monotone(h) and is_unbounded(h) and set_image(s, f).contains(c) and tendsto_metric(compose(q, h), c)
                }
            }
            not_forall_imp_exists_not[Nat -> N](function(q: Nat -> N) {
                (forall(n: Nat) { set_image(s, f).contains(q(n)) }) implies exists(h: Nat -> Nat, c: N) {
                    is_monotone(h) and is_unbounded(h) and set_image(s, f).contains(c) and tendsto_metric(compose(q, h), c)
                }
            })
            let q: Nat -> N satisfy {
                not ((forall(n: Nat) { set_image(s, f).contains(q(n)) }) implies exists(h: Nat -> Nat, c: N) {
                        is_monotone(h) and is_unbounded(h) and set_image(s, f).contains(c) and tendsto_metric(compose(q, h), c)
                    })
            }
            not_implies((forall(n: Nat) { set_image(s, f).contains(q(n)) }),
                exists(h: Nat -> Nat, c: N) {
                    is_monotone(h) and is_unbounded(h) and set_image(s, f).contains(c) and tendsto_metric(compose(q, h), c)
                })
            (not ((forall(n: Nat) { set_image(s, f).contains(q(n)) }) implies exists(h: Nat -> Nat, c: N) {
                    is_monotone(h) and is_unbounded(h) and set_image(s, f).contains(c) and tendsto_metric(compose(q, h), c)
                })) = ((forall(n: Nat) { set_image(s, f).contains(q(n)) }) and not exists(h: Nat -> Nat, c: N) {
                        is_monotone(h) and is_unbounded(h) and set_image(s, f).contains(c) and tendsto_metric(compose(q, h), c)
                    })
            (forall(n: Nat) { set_image(s, f).contains(q(n)) }) and not exists(h: Nat -> Nat, c: N) {
                    is_monotone(h) and is_unbounded(h) and set_image(s, f).contains(c) and tendsto_metric(compose(q, h), c)
                }
            forall(n: Nat) { set_image(s, f).contains(q(n)) }
            (forall(n: Nat) { set_image(s, f).contains(q(n)) }) implies exists(h: Nat -> Nat, c: N) {
                is_monotone(h) and is_unbounded(h) and set_image(s, f).contains(c) and tendsto_metric(compose(q, h), c)
            }
            exists(h: Nat -> Nat, c: N) {
                is_monotone(h) and is_unbounded(h) and set_image(s, f).contains(c) and tendsto_metric(compose(q, h), c)
            }
            not exists(h: Nat -> Nat, c: N) {
                is_monotone(h) and is_unbounded(h) and set_image(s, f).contains(c) and tendsto_metric(compose(q, h), c)
            }
            false
        }
        by_contradiction(is_sequentially_compact(set_image(s, f)))
        is_sequentially_compact(set_image(s, f))
    }
}

/// A sequentially compact set is bounded.
///
/// If `s` were unbounded, a choice sequence `q` with terms at distance at
/// least `n` from a fixed point would have no convergent subsequence: any
/// convergent subsequence would be bounded, but the subsequence of an
/// unbounded-index map is unbounded.
theorem sequentially_compact_imp_bounded[M: MetricSpace](s: Set[M]) {
    is_sequentially_compact(s) implies is_bounded(s)
} by {
    if is_sequentially_compact(s) {
        if not is_bounded(s) {
            metric_space_point_exists[M]
            let x0: M satisfy {
                true
            }
            forall(n: Nat) {
                is_bounded(s) = exists(c: M, r: Real) {
                    s.subset(closed_ball(c, r))
                }
                not exists(c: M, r: Real) {
                    s.subset(closed_ball(c, r))
                }
                s.subset(closed_ball(x0, from_nat[Real](n))) = forall(x: M) {
                    s.contains(x) implies closed_ball(x0, from_nat[Real](n)).contains(x)
                }
                not s.subset(closed_ball(x0, from_nat[Real](n)))
                not forall(x: M) {
                    s.contains(x) implies closed_ball(x0, from_nat[Real](n)).contains(x)
                }
                not_forall_imp_exists_not[M](function(x: M) {
                    s.contains(x) implies closed_ball(x0, from_nat[Real](n)).contains(x)
                })
                let x: M satisfy {
                    not (s.contains(x) implies closed_ball(x0, from_nat[Real](n)).contains(x))
                }
                not_implies(s.contains(x), closed_ball(x0, from_nat[Real](n)).contains(x))
                (not (s.contains(x) implies closed_ball(x0, from_nat[Real](n)).contains(x))) = (s.contains(x) and not closed_ball(x0, from_nat[Real](n)).contains(x))
                s.contains(x) and not closed_ball(x0, from_nat[Real](n)).contains(x)
                s.contains(x)
                not closed_ball(x0, from_nat[Real](n)).contains(x)
                closed_ball_contains_iff(x0, from_nat[Real](n), x)
                closed_ball(x0, from_nat[Real](n)).contains(x) = (x.distance(x0) <= from_nat[Real](n))
                not (x.distance(x0) <= from_nat[Real](n))
                not_lte_imp_gt(x.distance(x0), from_nat[Real](n))
                x.distance(x0) > from_nat[Real](n)
                from_nat[Real](n) < x.distance(x0)
                exists(y: M) {
                    s.contains(y) and from_nat[Real](n) < y.distance(x0)
                }
            }
            let q(n: Nat) -> result: M satisfy {
                s.contains(result) and from_nat[Real](n) < result.distance(x0)
            }
            forall(n: Nat) {
                s.contains(q(n))
            }
            sequentially_compact_subsequence(s, q)
            exists(f: Nat -> Nat, a: M) {
                is_monotone(f) and is_unbounded(f) and s.contains(a) and tendsto_metric(compose(q, f), a)
            }
            let f: Nat -> Nat satisfy {
                exists(a: M) {
                    is_monotone(f) and is_unbounded(f) and s.contains(a) and tendsto_metric(compose(q, f), a)
                }
            }
            let a: M satisfy {
                is_monotone(f) and is_unbounded(f) and s.contains(a) and tendsto_metric(compose(q, f), a)
            }
            is_unbounded(f)
            tendsto_metric(compose(q, f), a)
            tendsto_metric_seq_is_bounded(compose(q, f), a)
            is_bounded(sequence_range(compose(q, f)))
            is_bounded(sequence_range(compose(q, f))) = exists(c: M, r: Real) {
                sequence_range(compose(q, f)).subset(closed_ball(c, r))
            }
            let c: M satisfy {
                exists(r: Real) {
                    sequence_range(compose(q, f)).subset(closed_ball(c, r))
                }
            }
            let r: Real satisfy {
                sequence_range(compose(q, f)).subset(closed_ball(c, r))
            }
            real_lt_some_nat(r + c.distance(x0))
            let b: Nat satisfy {
                r + c.distance(x0) < from_nat[Real](b)
            }
            is_unbounded(f) = forall(bound: Nat) {
                exists(k: Nat) { bound < f(k) }
            }
            exists(k: Nat) { b < f(k) }
            let k: Nat satisfy {
                b < f(k)
            }
            lt_imp_lte(b, f(k))
            b <= f(k)
            from_nat_lte_mono(b, f(k))
            from_nat[Real](b) <= from_nat[Real](f(k))
            lt_lte_trans(r + c.distance(x0), from_nat[Real](b), from_nat[Real](f(k)))
            r + c.distance(x0) < from_nat[Real](f(k))
            from_nat[Real](f(k)) < q(f(k)).distance(x0)
            sequence_range_contains_eq(compose(q, f), q(f(k)))
            sequence_range(compose(q, f)).contains(q(f(k))) = exists(n: Nat) {
                compose(q, f)(n) = q(f(k))
            }
            compose(q, f)(k) = q(f(k))
            exists(n: Nat) {
                compose(q, f)(n) = q(f(k))
            }
            sequence_range(compose(q, f)).contains(q(f(k)))
            subset_contains(sequence_range(compose(q, f)), closed_ball(c, r), q(f(k)))
            closed_ball(c, r).contains(q(f(k)))
            closed_ball_contains_iff(c, r, q(f(k)))
            closed_ball(c, r).contains(q(f(k))) = (q(f(k)).distance(c) <= r)
            q(f(k)).distance(c) <= r
            q(f(k)).distance(x0) <= q(f(k)).distance(c) + c.distance(x0)
            add_le_add_right(q(f(k)).distance(c), r, c.distance(x0))
            q(f(k)).distance(c) + c.distance(x0) <= r + c.distance(x0)
            lte_trans(q(f(k)).distance(x0), q(f(k)).distance(c) + c.distance(x0), r + c.distance(x0))
            q(f(k)).distance(x0) <= r + c.distance(x0)
            lte_lt_trans(q(f(k)).distance(x0), r + c.distance(x0), from_nat[Real](f(k)))
            q(f(k)).distance(x0) < from_nat[Real](f(k))
            lt_trans(q(f(k)).distance(x0), from_nat[Real](f(k)), q(f(k)).distance(x0))
            q(f(k)).distance(x0) < q(f(k)).distance(x0)
            not_lt_self(q(f(k)).distance(x0))
            false
        }
        by_contradiction(is_bounded(s))
        is_bounded(s)
    }
}

/// A continuous real-valued function on a sequentially compact set is
/// bounded above.
///
/// The image is sequentially compact, hence bounded; a metric bound
/// `|f(y) - c| <= r` on the image gives the upper bound `c + r`.
theorem continuous_seq_compact_bounded_above[M: MetricTopologicalSpace](s: Set[M], f: M -> Real) {
    is_sequentially_compact(s) and is_continuous(f) implies exists(b: Real) {
        forall(y: M) { s.contains(y) implies f(y) <= b }
    }
} by {
    if is_sequentially_compact(s) and is_continuous(f) {
        sequentially_compact_image(s, f)
        is_sequentially_compact(set_image(s, f))
        sequentially_compact_imp_bounded(set_image(s, f))
        is_bounded(set_image(s, f))
        is_bounded(set_image(s, f)) = exists(c: Real, r: Real) {
            set_image(s, f).subset(closed_ball(c, r))
        }
        let c: Real satisfy {
            exists(r: Real) {
                set_image(s, f).subset(closed_ball(c, r))
            }
        }
        let r: Real satisfy {
            set_image(s, f).subset(closed_ball(c, r))
        }
        forall(y: M) {
            if s.contains(y) {
                set_image_contains_eq(s, f, f(y))
                set_image(s, f).contains(f(y)) = image_contains(f, s, f(y))
                image_contains(f, s, f(y)) = exists(x: M) {
                    s.contains(x) and f(y) = f(x)
                }
                s.contains(y) and f(y) = f(y)
                image_contains(f, s, f(y))
                set_image(s, f).contains(f(y))
                subset_contains(set_image(s, f), closed_ball(c, r), f(y))
                closed_ball(c, r).contains(f(y))
                closed_ball_contains_iff(c, r, f(y))
                closed_ball(c, r).contains(f(y)) = (f(y).distance(c) <= r)
                f(y).distance(c) <= r
                f(y).distance(c) = real_distance(f(y), c)
                real_distance(f(y), c) = (f(y) - c).abs
                (f(y) - c).abs <= r
                lte_abs(f(y) - c)
                f(y) - c <= (f(y) - c).abs
                lte_trans(f(y) - c, (f(y) - c).abs, r)
                f(y) - c <= r
                add_le_add_right(f(y) - c, r, c)
                (f(y) - c) + c <= r + c
                (f(y) - c) + c = f(y)
                f(y) <= r + c
                r + c = c + r
                f(y) <= c + r
            }
        }
        exists(b: Real) {
            forall(y: M) { s.contains(y) implies f(y) <= b }
        }
    }
}

/// A continuous real-valued function on a sequentially compact set is
/// bounded below.
///
/// A metric bound `|f(y) - c| <= r` on the image gives the lower bound
/// `c - r`.
theorem continuous_seq_compact_bounded_below[M: MetricTopologicalSpace](s: Set[M], f: M -> Real) {
    is_sequentially_compact(s) and is_continuous(f) implies exists(b: Real) {
        forall(y: M) { s.contains(y) implies b <= f(y) }
    }
} by {
    if is_sequentially_compact(s) and is_continuous(f) {
        sequentially_compact_image(s, f)
        is_sequentially_compact(set_image(s, f))
        sequentially_compact_imp_bounded(set_image(s, f))
        is_bounded(set_image(s, f))
        is_bounded(set_image(s, f)) = exists(c: Real, r: Real) {
            set_image(s, f).subset(closed_ball(c, r))
        }
        let c: Real satisfy {
            exists(r: Real) {
                set_image(s, f).subset(closed_ball(c, r))
            }
        }
        let r: Real satisfy {
            set_image(s, f).subset(closed_ball(c, r))
        }
        forall(y: M) {
            if s.contains(y) {
                set_image_contains_eq(s, f, f(y))
                set_image(s, f).contains(f(y)) = image_contains(f, s, f(y))
                image_contains(f, s, f(y)) = exists(x: M) {
                    s.contains(x) and f(y) = f(x)
                }
                s.contains(y) and f(y) = f(y)
                image_contains(f, s, f(y))
                set_image(s, f).contains(f(y))
                subset_contains(set_image(s, f), closed_ball(c, r), f(y))
                closed_ball(c, r).contains(f(y))
                closed_ball_contains_iff(c, r, f(y))
                closed_ball(c, r).contains(f(y)) = (f(y).distance(c) <= r)
                f(y).distance(c) <= r
                f(y).distance(c) = real_distance(f(y), c)
                real_distance(f(y), c) = (f(y) - c).abs
                (f(y) - c).abs <= r
                lte_abs(-(f(y) - c))
                -(f(y) - c) <= (-(f(y) - c)).abs
                abs_neg(f(y) - c)
                (-(f(y) - c)).abs = (f(y) - c).abs
                lte_trans(-(f(y) - c), (-(f(y) - c)).abs, r)
                -(f(y) - c) <= r
                neg_distrib(f(y), -c)
                -(f(y) + -c) = -f(y) + -(-c)
                neg_neg(c)
                -(-c) = c
                -f(y) + -(-c) = -f(y) + c
                f(y) - c = f(y) + -c
                -(f(y) - c) = -(f(y) + -c)
                -(f(y) - c) = -f(y) + c
                add_comm(-f(y), c)
                -f(y) + c = c + -f(y)
                c - f(y) = c + -f(y)
                -(f(y) - c) = c - f(y)
                c - f(y) <= r
                add_le_add_right(c - f(y), r, f(y))
                (c - f(y)) + f(y) <= r + f(y)
                (c - f(y)) + f(y) = c
                c <= r + f(y)
                add_le_add_right(c, r + f(y), -r)
                c + -r <= (r + f(y)) + -r
                c + -r = c - r
                add_assoc(r, f(y), -r)
                (r + f(y)) + -r = r + (f(y) + -r)
                add_comm(f(y), -r)
                f(y) + -r = -r + f(y)
                r + (f(y) + -r) = r + (-r + f(y))
                add_assoc(r, -r, f(y))
                (r + -r) + f(y) = r + (-r + f(y))
                add_neg_eq_zero(r)
                r + -r = Real.0
                (r + -r) + f(y) = Real.0 + f(y)
                add_zero_left(f(y))
                Real.0 + f(y) = f(y)
                (r + f(y)) + -r = f(y)
                c - r <= f(y)
            }
        }
        exists(b: Real) {
            forall(y: M) { s.contains(y) implies b <= f(y) }
        }
    }
}
