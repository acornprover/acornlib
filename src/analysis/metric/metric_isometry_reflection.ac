from nat import Nat
from real import Real
from data.basic.functions import compose
from analysis.metric.metric_space import MetricSpace, is_isometry, isometry_distance,
    metric_tail_bound, metric_tail_bound_at, metric_tail_bound_from_distances,
    cauchy_metric_bound, cauchy_metric_bound_at, cauchy_metric_bound_from_distances,
    tendsto_metric, tendsto_metric_from_tail_bounds,
    is_cauchy_metric, is_cauchy_metric_from_bounds

/// An isometry reflects a concrete metric tail bound from an image sequence.
theorem isometry_reflects_metric_tail_bound[M: MetricSpace, N: MetricSpace](
    f: M -> N, q: Nat -> M, a: M, n: Nat, eps: Real
) {
    is_isometry(f) and metric_tail_bound(compose(f, q), f(a), n, eps)
        implies metric_tail_bound(q, a, n, eps)
} by {
    if is_isometry(f) and metric_tail_bound(compose(f, q), f(a), n, eps) {
        forall(i: Nat) {
            if n <= i {
                metric_tail_bound_at(compose(f, q), f(a), n, eps, i)
                compose(f, q)(i).distance(f(a)) < eps
                compose(f, q, i) = f(q(i))
                f(q(i)).distance(f(a)) < eps
                isometry_distance(f, q(i), a)
                f(q(i)).distance(f(a)) = q(i).distance(a)
                q(i).distance(a) < eps
            }
        }
        metric_tail_bound_from_distances(q, a, n, eps)
        metric_tail_bound(q, a, n, eps)
    }
}

/// An isometry reflects metric convergence from an image sequence.
theorem isometry_reflects_tendsto_metric[M: MetricSpace, N: MetricSpace](
    f: M -> N, q: Nat -> M, a: M
) {
    is_isometry(f) and tendsto_metric(compose(f, q), f(a)) implies tendsto_metric(q, a)
} by {
    if is_isometry(f) and tendsto_metric(compose(f, q), f(a)) {
        forall(eps: Real) {
            if eps.is_positive {
                tendsto_metric(compose(f, q), f(a)) = forall(e: Real) {
                    e.is_positive implies exists(n: Nat) {
                        metric_tail_bound(compose(f, q), f(a), n, e)
                    }
                }
                let n: Nat satisfy {
                    metric_tail_bound(compose(f, q), f(a), n, eps)
                }
                isometry_reflects_metric_tail_bound(f, q, a, n, eps)
                metric_tail_bound(q, a, n, eps)
                exists(m: Nat) { metric_tail_bound(q, a, m, eps) }
            }
        }
        forall(eps: Real) {
            eps.is_positive implies exists(n: Nat) {
                metric_tail_bound(q, a, n, eps)
            }
        }
    }
}

/// An isometry reflects a concrete metric Cauchy bound from an image sequence.
theorem isometry_reflects_cauchy_metric_bound[M: MetricSpace, N: MetricSpace](
    f: M -> N, q: Nat -> M, n: Nat, eps: Real
) {
    is_isometry(f) and cauchy_metric_bound(compose(f, q), n, eps)
        implies cauchy_metric_bound(q, n, eps)
} by {
    if is_isometry(f) and cauchy_metric_bound(compose(f, q), n, eps) {
        forall(i: Nat, j: Nat) {
            if n <= i and n <= j {
                cauchy_metric_bound_at(compose(f, q), n, eps, i, j)
                compose(f, q)(i).distance(compose(f, q)(j)) < eps
                compose(f, q, i) = f(q(i))
                compose(f, q, j) = f(q(j))
                f(q(i)).distance(f(q(j))) < eps
                isometry_distance(f, q(i), q(j))
                f(q(i)).distance(f(q(j))) = q(i).distance(q(j))
                q(i).distance(q(j)) < eps
            }
        }
        cauchy_metric_bound_from_distances(q, n, eps)
        cauchy_metric_bound(q, n, eps)
    }
}

/// An isometry reflects metric Cauchy sequences from image sequences.
theorem isometry_reflects_cauchy_metric[M: MetricSpace, N: MetricSpace](
    f: M -> N, q: Nat -> M
) {
    is_isometry(f) and is_cauchy_metric(compose(f, q)) implies is_cauchy_metric(q)
} by {
    if is_isometry(f) and is_cauchy_metric(compose(f, q)) {
        forall(eps: Real) {
            if eps.is_positive {
                is_cauchy_metric(compose(f, q)) = forall(e: Real) {
                    e.is_positive implies exists(n: Nat) {
                        cauchy_metric_bound(compose(f, q), n, e)
                    }
                }
                let n: Nat satisfy {
                    cauchy_metric_bound(compose(f, q), n, eps)
                }
                isometry_reflects_cauchy_metric_bound(f, q, n, eps)
                cauchy_metric_bound(q, n, eps)
                exists(m: Nat) { cauchy_metric_bound(q, m, eps) }
            }
        }
        forall(eps: Real) {
            eps.is_positive implies exists(n: Nat) {
                cauchy_metric_bound(q, n, eps)
            }
        }
    }
}
