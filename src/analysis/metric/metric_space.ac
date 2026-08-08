from nat import Nat
from order import lte_trans, lte_lt_trans, lt_lte_trans
from real import lt_add_converse, lte_mul_nonneg_right, lte_mul_nonneg_left, exists_small_mul
from real import Real
from data.basic.functions import compose, identity_fn, is_injective_fn, compose_injective_fn
from real import eps_lt_half, lt_imp_minus_pos
from real import completeness, has_upper_bound, is_nonempty, is_set_supremum, is_set_upper_bound, supremum_subset_le
from data.basic.set import Set, subset_contains, set_eq_of_contains_at_eq, set_image, set_image_contains_witness
numerals Real

/// A metric space is a set equipped with a distance function that satisfies specific axioms.
/// The distance function must be non-negative, symmetric, and satisfy the triangle inequality.
typeclass M: MetricSpace {
    /// Every metric space must have a distance function
    distance: (M, M) -> Real

    /// Rule: the distance from a point to itself is zero
    self_distance_is_zero(x: M) {
        x.distance(x) = 0
    }

    /// Rule: distance zero means the points are equal
    dist_zero_imp_eq(x: M, y: M) {
        x.distance(y) = 0 implies x = y
    }

    /// Rule: distance must be symmetric
    symmetric(x: M, y: M) {
        x.distance(y) = y.distance(x)
    }

    /// Rule: distance must satisfy the triangle inequality
    triangle(x: M, y: M, z: M) {
        x.distance(z) <= x.distance(y) + y.distance(z)
    }
}

theorem distance_non_negative[M: MetricSpace](x: M, y: M) {
    not x.distance(y).is_negative
} by {
    0 <= x.distance(y) + x.distance(y)
}

theorem distance_self[M: MetricSpace](x: M) {
    x.distance(x) = 0
}

theorem eq_imp_distance_zero[M: MetricSpace](x: M, y: M) {
    x = y implies x.distance(y) = 0
} by {
    if x = y {
        x.distance(x) = 0
    }
}

theorem distance_zero_iff_eq[M: MetricSpace](x: M, y: M) {
    (x.distance(y) = 0) = (x = y)
} by {
    if x.distance(y) = 0 {
        x = y
    }
    if x = y {
        x.distance(y) = 0
    }
}

/// True if `y` is strictly less than `r` away from `x` in the metric space `M`.
define in_open_ball[M: MetricSpace](x: M, r: Real, y: M) -> Bool {
    y.distance(x) < r
}

/// The open ball of radius `r` centered at `x`.
define open_ball[M: MetricSpace](x: M, r: Real) -> Set[M] {
    Set[M].new(in_open_ball(x, r))
}

/// True if `y` is at most `r` away from `x` in the metric space `M`.
define in_closed_ball[M: MetricSpace](x: M, r: Real, y: M) -> Bool {
    y.distance(x) <= r
}

/// The closed ball of radius `r` centered at `x`.
define closed_ball[M: MetricSpace](x: M, r: Real) -> Set[M] {
    Set[M].new(in_closed_ball(x, r))
}

theorem open_ball_contains_iff[M: MetricSpace](x: M, r: Real, y: M) {
    open_ball(x, r).contains(y) = (y.distance(x) < r)
}

theorem closed_ball_contains_iff[M: MetricSpace](x: M, r: Real, y: M) {
    closed_ball(x, r).contains(y) = (y.distance(x) <= r)
}

theorem center_in_open_ball[M: MetricSpace](x: M, r: Real) {
    r.is_positive implies open_ball(x, r).contains(x)
} by {
    if r.is_positive {
        0 < r
        x.distance(x) < r
    }
}

theorem center_in_closed_ball[M: MetricSpace](x: M, r: Real) {
    not r.is_negative implies closed_ball(x, r).contains(x)
} by {
    if not r.is_negative {
        x.distance(x) <= r
    }
}

theorem open_ball_subset_closed_ball[M: MetricSpace](x: M, r: Real) {
    open_ball(x, r).subset(closed_ball(x, r))
} by {
    forall(y: M) {
        if open_ball(x, r).contains(y) {
            y.distance(x) < r
            y.distance(x) <= r
            closed_ball(x, r).contains(y)
        }
    }
}

/// Open balls are monotone in the radius.
theorem open_ball_subset_of_radius_le[M: MetricSpace](x: M, r1: Real, r2: Real) {
    r1 <= r2 implies open_ball(x, r1).subset(open_ball(x, r2))
} by {
    if r1 <= r2 {
        forall(y: M) {
            if open_ball(x, r1).contains(y) {
                lt_lte_trans(y.distance(x), r1, r2)
                y.distance(x) < r2
                open_ball(x, r2).contains(y)
            }
        }
    }
}

/// Closed balls are monotone in the radius.
theorem closed_ball_subset_of_radius_le[M: MetricSpace](x: M, r1: Real, r2: Real) {
    r1 <= r2 implies closed_ball(x, r1).subset(closed_ball(x, r2))
} by {
    if r1 <= r2 {
        forall(y: M) {
            if closed_ball(x, r1).contains(y) {
                lte_trans(y.distance(x), r1, r2)
                y.distance(x) <= r2
                closed_ball(x, r2).contains(y)
            }
        }
    }
}

theorem open_ball_nonpositive_radius_empty[M: MetricSpace](x: M, r: Real, y: M) {
    not r.is_positive implies not open_ball(x, r).contains(y)
} by {
    if not r.is_positive {
        not y.distance(x).is_negative
        if open_ball(x, r).contains(y) {
            y.distance(x) < r
            false
        }
    }
}

/// An open ball of nonpositive radius has no points.
theorem open_ball_nonpositive_radius_is_empty[M: MetricSpace](x: M, r: Real) {
    not r.is_positive implies open_ball(x, r).is_empty
} by {
    if not r.is_positive {
        forall(y: M) {
            open_ball_nonpositive_radius_empty(x, r, y)
            not open_ball(x, r).contains(y)
        }
    }
}

/// A closed ball of negative radius has no points.
theorem closed_ball_negative_radius_empty[M: MetricSpace](x: M, r: Real, y: M) {
    r.is_negative implies not closed_ball(x, r).contains(y)
} by {
    if r.is_negative {
        r < 0
        if closed_ball(x, r).contains(y) {
            closed_ball(x, r).contains(y) = (y.distance(x) <= r)
            y.distance(x) <= r
            lte_lt_trans(y.distance(x), r, 0)
            y.distance(x) < 0
            distance_non_negative(y, x)
            not y.distance(x).is_negative
            false
        }
    }
}

/// A closed ball of negative radius is empty.
theorem closed_ball_negative_radius_is_empty[M: MetricSpace](x: M, r: Real) {
    r.is_negative implies closed_ball(x, r).is_empty
} by {
    if r.is_negative {
        forall(y: M) {
            closed_ball_negative_radius_empty(x, r, y)
            not closed_ball(x, r).contains(y)
        }
    }
}

/// True if `y` is strictly less than `r` away from `x` is symmetric in `x` and `y`.
theorem in_open_ball_symmetric[M: MetricSpace](x: M, r: Real, y: M) {
    open_ball(x, r).contains(y) = open_ball(y, r).contains(x)
} by {
    x.distance(y) = y.distance(x)
}

/// True if `y` is at most `r` away from `x` is symmetric in `x` and `y`.
theorem in_closed_ball_symmetric[M: MetricSpace](x: M, r: Real, y: M) {
    closed_ball(x, r).contains(y) = closed_ball(y, r).contains(x)
} by {
    x.distance(y) = y.distance(x)
}

/// True if there is a closed ball that contains the entire set `s`.
define is_bounded[M: MetricSpace](s: Set[M]) -> Bool {
    exists(x: M, r: Real) {
        s.subset(closed_ball(x, r))
    }
}

/// The empty set is bounded in every metric space.
theorem empty_set_is_bounded[M: MetricSpace](x: M) {
    is_bounded(Set[M].empty_set)
} by {
    forall(y: M) {
        not (Set[M].empty_set).contains(y)
    }
}

/// A subset of a bounded set is bounded.
theorem subset_of_bounded_is_bounded[M: MetricSpace](s: Set[M], t: Set[M]) {
    s.subset(t) and is_bounded(t) implies is_bounded(s)
} by {
    if s.subset(t) and is_bounded(t) {
        let x: M satisfy {
            exists(r: Real) { t.subset(closed_ball(x, r)) }
        }
        let r: Real satisfy {
            t.subset(closed_ball(x, r))
        }
        forall(y: M) {
            if s.contains(y) {
                t.contains(y)
                closed_ball(x, r).contains(y)
            }
        }
        s.subset(closed_ball(x, r))
        is_bounded(s)
    }
}

/// A singleton set is bounded.
theorem singleton_is_bounded[M: MetricSpace](x: M) {
    is_bounded(Set[M].singleton(x))
} by {
    forall(y: M) {
        if (Set[M].singleton(x)).contains(y) {
            (Set[M].singleton(x)).contains(y) = (x = y)
            x = y
            y.distance(x) = 0
            closed_ball(x, 0).contains(y)
        }
    }
    (Set[M].singleton(x)).subset(closed_ball(x, 0))
}

/// The union of two bounded sets is bounded.
theorem union_of_bounded_is_bounded[M: MetricSpace](s: Set[M], t: Set[M]) {
    is_bounded(s) and is_bounded(t) implies is_bounded(s.union(t))
} by {
    if is_bounded(s) and is_bounded(t) {
        let x1: M satisfy {
            exists(r: Real) { s.subset(closed_ball(x1, r)) }
        }
        let r1: Real satisfy {
            s.subset(closed_ball(x1, r1))
        }
        let x2: M satisfy {
            exists(r: Real) { t.subset(closed_ball(x2, r)) }
        }
        let r2: Real satisfy {
            t.subset(closed_ball(x2, r2))
        }
        let r: Real = r1.abs + x1.distance(x2) + r2.abs
        not r1.abs.is_negative
        not r2.abs.is_negative
        not x1.distance(x2).is_negative
        forall(y: M) {
            if s.union(t).contains(y) {
                s.contains(y) or t.contains(y)
                if s.contains(y) {
                    closed_ball(x1, r1).contains(y)
                    y.distance(x1) <= r1
                    r1 <= r1.abs
                    y.distance(x1) <= r1.abs
                    let m1: Real = r1.abs + x1.distance(x2)
                    Real.0 + r1.abs = r1.abs
                    Real.0 + r1.abs <= x1.distance(x2) + r1.abs
                    r1.abs <= x1.distance(x2) + r1.abs
                    x1.distance(x2) + r1.abs = m1
                    r1.abs <= m1
                    Real.0 + m1 = m1
                    Real.0 + m1 <= r2.abs + m1
                    m1 <= r2.abs + m1
                    r2.abs + m1 = r
                    r1.abs <= r
                    y.distance(x1) <= r
                    closed_ball(x1, r).contains(y)
                } else {
                    t.contains(y)
                    closed_ball(x2, r2).contains(y)
                    y.distance(x2) <= r2
                    r2 <= r2.abs
                    y.distance(x2) <= r2.abs
                    y.distance(x1) <= y.distance(x2) + x2.distance(x1)
                    x2.distance(x1) = x1.distance(x2)
                    y.distance(x2) + x2.distance(x1) <= r2.abs + x1.distance(x2)
                    y.distance(x1) <= r2.abs + x1.distance(x2)
                    let m2: Real = r2.abs + x1.distance(x2)
                    y.distance(x1) <= m2
                    Real.0 + m2 = m2
                    Real.0 + m2 <= r1.abs + m2
                    m2 <= r1.abs + m2
                    r1.abs + m2 = r
                    y.distance(x1) <= r
                    closed_ball(x1, r).contains(y)
                }
            }
        }
        s.union(t).subset(closed_ball(x1, r))
        is_bounded(s.union(t))
    }
}

/// A closed ball is bounded.
theorem closed_ball_is_bounded[M: MetricSpace](x: M, r: Real) {
    is_bounded(closed_ball(x, r))
} by {
    closed_ball(x, r).subset(closed_ball(x, r))
}

/// An open ball is bounded.
theorem open_ball_is_bounded[M: MetricSpace](x: M, r: Real) {
    is_bounded(open_ball(x, r))
} by {
    open_ball_subset_closed_ball(x, r)
}

/// The discrete metric assigns distance 0 to identical elements and distance 1 to distinct elements.
define discrete[T](x: T, y: T) -> Real {
    if x = y {
        0
    } else {
        1
    }
}

theorem zero_lte_discrete[T](x: T, y: T) {
    0 <= discrete(x, y)
} by {
    if x = y {
        discrete(x, y) = 0
        0 <= discrete(x, y)
    } else {
        discrete(x, y) = 1
        0 <= 1
        0 <= discrete(x, y)
    }
}

theorem discrete_self_distance_is_zero[T](x: T) {
    discrete(x, x) = 0
}

theorem discrete_dist_zero_imp_eq[T](x: T, y: T) {
    discrete(x, y) = 0 implies x = y
}

theorem discrete_eq_one_of_ne[T](x: T, y: T) {
    x != y implies discrete(x, y) = 1
} by {
    if x != y {
        if x = y {
            false
        } else {
            discrete(x, y) = 1
        }
    }
}

theorem discrete_symmetric[T](x: T, y: T) {
    discrete(x, y) = discrete(y, x)
} by {
    if x = y {
    } else {
        discrete(x, y) = discrete(y, x)
    }
}

theorem discrete_triangle[T](x: T, y: T, z: T) {
    discrete(x, z) <= discrete(x, y) + discrete(y, z)
} by {
    if x = y {
    } else {
        0 <= discrete(y, z)
        if x = z {
            discrete(x, z) <= discrete(x, y) + discrete(y, z)
        } else {
            discrete(x, z) <= discrete(x, y) + discrete(y, z)
        }
    }
}

/// A simple enumeration type used as an example of a metric space with the discrete metric.
inductive Color {
    /// Red means nothing. It's just different from green and blue.
    red

    /// Green is not red or blue.
    green

    /// Blue is not red or green.
    blue
}

instance Color: MetricSpace {
    let distance: (Color, Color) -> Real = discrete
}

/// True if every term `q(i)` with `i >= n` is within `eps` of the limit candidate `a`.
define metric_tail_bound[M: MetricSpace](q: Nat -> M, a: M, n: Nat, eps: Real) -> Bool {
    forall(i: Nat) {
        n <= i implies q(i).distance(a) < eps
    }
}

/// A sequence `q` in a metric space tends to `a` if for every positive `eps`,
/// some tail of `q` lies within `eps` of `a`.
define tendsto_metric[M: MetricSpace](q: Nat -> M, a: M) -> Bool {
    forall(eps: Real) {
        eps.is_positive implies exists(n: Nat) {
            metric_tail_bound(q, a, n, eps)
        }
    }
}

/// True if every pair of terms `q(i)`, `q(j)` with `i, j >= n` are within `eps` of each other.
define cauchy_metric_bound[M: MetricSpace](q: Nat -> M, n: Nat, eps: Real) -> Bool {
    forall(i: Nat, j: Nat) {
        n <= i and n <= j implies q(i).distance(q(j)) < eps
    }
}

/// A sequence `q` in a metric space is Cauchy if for every positive `eps`,
/// some tail of `q` has all pairs within `eps`.
define is_cauchy_metric[M: MetricSpace](q: Nat -> M) -> Bool {
    forall(eps: Real) {
        eps.is_positive implies exists(n: Nat) {
            cauchy_metric_bound(q, n, eps)
        }
    }
}

/// Unpacking `tendsto_metric` at a specific positive `eps`.
theorem tendsto_metric_tail_bound[M: MetricSpace](q: Nat -> M, a: M, eps: Real) {
    tendsto_metric(q, a) and eps.is_positive implies exists(n: Nat) {
        metric_tail_bound(q, a, n, eps)
    }
} by {
    tendsto_metric(q, a) = forall(e: Real) {
        e.is_positive implies exists(n: Nat) {
            metric_tail_bound(q, a, n, e)
        }
    }
}

/// Unpacking `metric_tail_bound` at a specific index.
theorem metric_tail_bound_at[M: MetricSpace](q: Nat -> M, a: M, n: Nat, eps: Real, i: Nat) {
    metric_tail_bound(q, a, n, eps) and n <= i implies q(i).distance(a) < eps
} by {
    metric_tail_bound(q, a, n, eps) = forall(j: Nat) {
        n <= j implies q(j).distance(a) < eps
    }
}

/// Build a metric tail bound from a pointwise distance bound.
theorem metric_tail_bound_from_distances[M: MetricSpace](q: Nat -> M, a: M, n: Nat, eps: Real) {
    (forall(i: Nat) {
        n <= i implies q(i).distance(a) < eps
    }) implies metric_tail_bound(q, a, n, eps)
} by {
    if forall(i: Nat) {
        n <= i implies q(i).distance(a) < eps
    } {
        metric_tail_bound(q, a, n, eps) = forall(i: Nat) {
            n <= i implies q(i).distance(a) < eps
        }
    }
}

/// Build a Cauchy metric bound from a pointwise distance bound.
theorem cauchy_metric_bound_from_distances[M: MetricSpace](q: Nat -> M, n: Nat, eps: Real) {
    (forall(i: Nat, j: Nat) {
        n <= i and n <= j implies q(i).distance(q(j)) < eps
    }) implies cauchy_metric_bound(q, n, eps)
} by {
    if forall(i: Nat, j: Nat) {
        n <= i and n <= j implies q(i).distance(q(j)) < eps
    } {
        cauchy_metric_bound(q, n, eps) = forall(i: Nat, j: Nat) {
            n <= i and n <= j implies q(i).distance(q(j)) < eps
        }
    }
}

/// Unpacking `cauchy_metric_bound` at a specific pair of indices.
theorem cauchy_metric_bound_at[M: MetricSpace](q: Nat -> M, n: Nat, eps: Real, i: Nat, j: Nat) {
    cauchy_metric_bound(q, n, eps) and n <= i and n <= j implies q(i).distance(q(j)) < eps
} by {
    cauchy_metric_bound(q, n, eps) = forall(a: Nat, b: Nat) {
        n <= a and n <= b implies q(a).distance(q(b)) < eps
    }
}

/// Unpacking `is_cauchy_metric` at a specific positive `eps`.
theorem is_cauchy_metric_bound[M: MetricSpace](q: Nat -> M, eps: Real) {
    is_cauchy_metric(q) and eps.is_positive implies exists(n: Nat) {
        cauchy_metric_bound(q, n, eps)
    }
} by {
    if is_cauchy_metric(q) and eps.is_positive {
        is_cauchy_metric(q) = forall(e: Real) {
            e.is_positive implies exists(n: Nat) {
                cauchy_metric_bound(q, n, e)
            }
        }
        exists(n: Nat) {
            cauchy_metric_bound(q, n, eps)
        }
    }
}

/// A sequence tends to a point when every positive radius has a bounded tail.
theorem tendsto_metric_from_tail_bounds[M: MetricSpace](q: Nat -> M, a: M) {
    (forall(eps: Real) {
        eps.is_positive implies exists(n: Nat) {
            metric_tail_bound(q, a, n, eps)
        }
    }) implies tendsto_metric(q, a)
} by {
    if forall(eps: Real) {
        eps.is_positive implies exists(n: Nat) {
            metric_tail_bound(q, a, n, eps)
        }
    } {
        tendsto_metric(q, a) = forall(eps: Real) {
            eps.is_positive implies exists(n: Nat) {
                metric_tail_bound(q, a, n, eps)
            }
        }
        tendsto_metric(q, a)
    }
}

/// A sequence is Cauchy when every positive radius has a pairwise bounded tail.
theorem is_cauchy_metric_from_bounds[M: MetricSpace](q: Nat -> M) {
    (forall(eps: Real) {
        eps.is_positive implies exists(n: Nat) {
            cauchy_metric_bound(q, n, eps)
        }
    }) implies is_cauchy_metric(q)
} by {
    if forall(eps: Real) {
        eps.is_positive implies exists(n: Nat) {
            cauchy_metric_bound(q, n, eps)
        }
    } {
        is_cauchy_metric(q) = forall(eps: Real) {
            eps.is_positive implies exists(n: Nat) {
                cauchy_metric_bound(q, n, eps)
            }
        }
        is_cauchy_metric(q)
    }
}

/// A convergent sequence in a metric space is Cauchy.
theorem tendsto_metric_imp_cauchy[M: MetricSpace](q: Nat -> M, a: M) {
    tendsto_metric(q, a) implies is_cauchy_metric(q)
} by {
    if tendsto_metric(q, a) {
        forall(eps: Real) {
            if eps.is_positive {
                let half: Real satisfy {
                    half.is_positive and half + half < eps
                }
                tendsto_metric_tail_bound(q, a, half)
                let n: Nat satisfy {
                    metric_tail_bound(q, a, n, half)
                }
                forall(i: Nat, j: Nat) {
                    if n <= i and n <= j {
                        metric_tail_bound_at(q, a, n, half, i)
                        metric_tail_bound_at(q, a, n, half, j)
                        q(i).distance(a) < half
                        q(j).distance(a) < half
                        q(i).distance(q(j)) <= q(i).distance(a) + a.distance(q(j))
                        a.distance(q(j)) = q(j).distance(a)
                        q(i).distance(a) + a.distance(q(j)) = q(i).distance(a) + q(j).distance(a)
                        q(i).distance(a) + q(j).distance(a) < half + half
                        q(i).distance(q(j)) < half + half
                        q(i).distance(q(j)) < eps
                    }
                }
                cauchy_metric_bound(q, n, eps)
                exists(m: Nat) { cauchy_metric_bound(q, m, eps) }
            }
        }
    }
}

/// True if `r` arises as the distance between two points of `s`.
define distance_set_contains[M: MetricSpace](s: Set[M], r: Real) -> Bool {
    exists(x: M, y: M) {
        s.contains(x) and s.contains(y) and r = x.distance(y)
    }
}

/// The set of pairwise distances of points in `s`.
define distance_set[M: MetricSpace](s: Set[M]) -> Set[Real] {
    Set[Real].new(distance_set_contains(s))
}

/// Membership in `distance_set` is membership in the pairwise-distance predicate.
theorem distance_set_contains_iff[M: MetricSpace](s: Set[M], r: Real) {
    distance_set(s).contains(r) = distance_set_contains(s, r)
}

/// Membership of a concrete pairwise distance in `distance_set`.
theorem distance_set_contains_distance[M: MetricSpace](s: Set[M], x: M, y: M) {
    s.contains(x) and s.contains(y) implies distance_set(s).contains(x.distance(y))
} by {
    if s.contains(x) and s.contains(y) {
        x.distance(y) = x.distance(y)
        distance_set_contains(s, x.distance(y))
    }
}

/// True if `d` is the supremum of all pairwise distances in `s`.
define is_diameter[M: MetricSpace](s: Set[M], d: Real) -> Bool {
    is_set_supremum(distance_set(s), d)
}

/// True if a set has a finite diameter: it is bounded and nonempty.
define has_diameter[M: MetricSpace](s: Set[M]) -> Bool {
    is_bounded(s) and exists(x: M) { s.contains(x) }
}

/// If `s` is bounded then `2 * r` is an upper bound on the pairwise distances,
/// where `r` is any radius witnessing the bound.
theorem bounded_distance_upper_bound[M: MetricSpace](s: Set[M], c: M, r: Real) {
    s.subset(closed_ball(c, r)) implies is_set_upper_bound(distance_set(s), r + r)
} by {
    if s.subset(closed_ball(c, r)) {
        forall(d: Real) {
            if distance_set(s).contains(d) {
                distance_set_contains(s, d)
                let x: M satisfy {
                    exists(y: M) {
                        s.contains(x) and s.contains(y) and d = x.distance(y)
                    }
                }
                let y: M satisfy {
                    s.contains(x) and s.contains(y) and d = x.distance(y)
                }
                closed_ball(c, r).contains(x)
                closed_ball(c, r).contains(y)
                x.distance(c) <= r
                y.distance(c) <= r
                x.distance(y) <= x.distance(c) + c.distance(y)
                c.distance(y) = y.distance(c)
                x.distance(c) + c.distance(y) = x.distance(c) + y.distance(c)
                x.distance(c) + y.distance(c) <= r + r
                x.distance(y) <= r + r
                d <= r + r
            }
        }
        is_set_upper_bound(distance_set(s), r + r)
    }
}

/// A bounded nonempty set has a diameter.
theorem diameter_exists[M: MetricSpace](s: Set[M]) {
    is_bounded(s) and exists(x: M) { s.contains(x) }
    implies
    exists(d: Real) { is_diameter(s, d) }
} by {
    if is_bounded(s) and exists(x: M) { s.contains(x) } {
        let x0: M satisfy { s.contains(x0) }
        distance_set_contains_distance(s, x0, x0)
        is_nonempty(distance_set(s))
        let c: M satisfy {
            exists(r: Real) { s.subset(closed_ball(c, r)) }
        }
        let r: Real satisfy {
            s.subset(closed_ball(c, r))
        }
        bounded_distance_upper_bound(s, c, r)
        has_upper_bound(distance_set(s))
        completeness(distance_set(s))
        let d: Real satisfy {
            is_set_supremum(distance_set(s), d)
        }
        is_diameter(s, d)
    }
}

/// In a set with a diameter, any pairwise distance is at most the diameter.
theorem distance_le_diameter[M: MetricSpace](s: Set[M], d: Real, x: M, y: M) {
    is_diameter(s, d) and s.contains(x) and s.contains(y)
    implies x.distance(y) <= d
} by {
    if is_diameter(s, d) and s.contains(x) and s.contains(y) {
        is_set_upper_bound(distance_set(s), d) = forall(z: Real) {
            distance_set(s).contains(z) implies z <= d
        }
        is_set_upper_bound(distance_set(s), d)
        distance_set_contains_distance(s, x, y)
        x.distance(y) <= d
    }
}

/// The diameter of a bounded nonempty set is non-negative.
theorem diameter_non_negative[M: MetricSpace](s: Set[M], d: Real, x: M) {
    is_diameter(s, d) and s.contains(x) implies 0 <= d
} by {
    if is_diameter(s, d) and s.contains(x) {
        distance_le_diameter(s, d, x, x)
        x.distance(x) = 0
        0 <= d
    }
}

/// The diameter is unique when it exists.
theorem diameter_unique[M: MetricSpace](s: Set[M], d1: Real, d2: Real) {
    is_diameter(s, d1) and is_diameter(s, d2) implies d1 = d2
} by {
    if is_diameter(s, d1) and is_diameter(s, d2) {
        let ds: Set[Real] = distance_set(s)
        is_set_supremum(ds, d1)
        is_set_supremum(ds, d2)
        is_set_supremum(ds, d1) = (is_set_upper_bound(ds, d1) and
            forall(b: Real) {
                is_set_upper_bound(ds, b) implies d1 <= b
            })
        is_set_upper_bound(ds, d1)
        is_set_upper_bound(ds, d2)
        forall(b: Real) {
            is_set_upper_bound(ds, b) implies d1 <= b
        }
        forall(b: Real) {
            is_set_upper_bound(ds, b) implies d2 <= b
        }
        d1 <= d2
        d2 <= d1
        d1 = d2
    }
}

/// Limits of convergent sequences in a metric space are unique.
theorem tendsto_metric_unique[M: MetricSpace](q: Nat -> M, a: M, b: M) {
    tendsto_metric(q, a) and tendsto_metric(q, b) implies a = b
} by {
    if tendsto_metric(q, a) and tendsto_metric(q, b) {
        forall(eps: Real) {
            if eps.is_positive {
                let half: Real satisfy {
                    half.is_positive and half + half < eps
                }
                tendsto_metric_tail_bound(q, a, half)
                tendsto_metric_tail_bound(q, b, half)
                let na: Nat satisfy {
                    metric_tail_bound(q, a, na, half)
                }
                let nb: Nat satisfy {
                    metric_tail_bound(q, b, nb, half)
                }
                let n: Nat = if na <= nb { nb } else { na }
                na <= n
                nb <= n
                metric_tail_bound_at(q, a, na, half, n)
                metric_tail_bound_at(q, b, nb, half, n)
                q(n).distance(a) < half
                q(n).distance(b) < half
                a.distance(b) <= a.distance(q(n)) + q(n).distance(b)
                a.distance(q(n)) = q(n).distance(a)
                a.distance(q(n)) + q(n).distance(b) < half + half
                a.distance(b) < half + half
                half + half < eps
                a.distance(b) < eps
            }
        }
        // a.distance(b) is non-negative and strictly less than every positive eps,
        // hence equals zero, hence a = b.
        if a.distance(b) != 0 {
            a.distance(b).is_positive
            let eps = a.distance(b)
            a.distance(b) < eps
            false
        }
        a = b
    }
}

/// True if `f` satisfies the Lipschitz distance inequality for the constant `k`.
define lipschitz_bound[M: MetricSpace, N: MetricSpace](k: Real, f: M -> N) -> Bool {
    forall(x: M, y: M) {
        f(x).distance(f(y)) <= k * x.distance(y)
    }
}

/// True if `k` is a Lipschitz constant for `f`.
define is_lipschitz_with[M: MetricSpace, N: MetricSpace](k: Real, f: M -> N) -> Bool {
    not k.is_negative and lipschitz_bound(k, f)
}

/// True if `f` has some Lipschitz constant.
define is_lipschitz[M: MetricSpace, N: MetricSpace](f: M -> N) -> Bool {
    exists(k: Real) {
        is_lipschitz_with(k, f)
    }
}

/// A Lipschitz constant is non-negative.
theorem lipschitz_with_constant_non_negative[M: MetricSpace, N: MetricSpace](k: Real, f: M -> N) {
    is_lipschitz_with(k, f) implies not k.is_negative
} by {
    if is_lipschitz_with(k, f) {
        not k.is_negative
    }
}

/// A Lipschitz map satisfies its distance bound at each pair of points.
theorem lipschitz_with_distance[M: MetricSpace, N: MetricSpace](k: Real, f: M -> N, x: M, y: M) {
    is_lipschitz_with(k, f) implies f(x).distance(f(y)) <= k * x.distance(y)
} by {
    if is_lipschitz_with(k, f) {
        lipschitz_bound(k, f)
        lipschitz_bound(k, f) = forall(a: M, b: M) {
            f(a).distance(f(b)) <= k * a.distance(b)
        }
        f(x).distance(f(y)) <= k * x.distance(y)
    }
}

/// A Lipschitz map with constant at most one is nonexpansive at each pair of points.
theorem lipschitz_with_le_one_distance[M: MetricSpace, N: MetricSpace](k: Real, f: M -> N, x: M, y: M) {
    is_lipschitz_with(k, f) and k <= 1 implies f(x).distance(f(y)) <= x.distance(y)
} by {
    if is_lipschitz_with(k, f) and k <= 1 {
        lipschitz_with_distance(k, f, x, y)
        f(x).distance(f(y)) <= k * x.distance(y)
        distance_non_negative(x, y)
        lte_mul_nonneg_right(k, 1, x.distance(y))
        lte_trans(f(x).distance(f(y)), k * x.distance(y), x.distance(y))
        f(x).distance(f(y)) <= x.distance(y)
    }
}

/// A map has the Lipschitz bound when the distance inequality holds for every pair.
theorem lipschitz_bound_from_distances[M: MetricSpace, N: MetricSpace](k: Real, f: M -> N) {
    (forall(x: M, y: M) {
        f(x).distance(f(y)) <= k * x.distance(y)
    }) implies lipschitz_bound(k, f)
} by {
    if forall(x: M, y: M) {
        f(x).distance(f(y)) <= k * x.distance(y)
    } {
        lipschitz_bound(k, f)
    }
}

/// The identity map is Lipschitz with constant one.
theorem identity_is_lipschitz_with_one[M: MetricSpace] {
    is_lipschitz_with(1, identity_fn[M])
} by {
    not Real.1.is_negative
    forall(x: M, y: M) {
        identity_fn[M](x).distance(identity_fn[M](y)) <= Real.1 * x.distance(y)
    }
    lipschitz_bound_from_distances[M, M](Real.1, identity_fn[M])
}

/// The identity map is Lipschitz.
theorem identity_is_lipschitz[M: MetricSpace] {
    is_lipschitz(identity_fn[M])
} by {
    identity_is_lipschitz_with_one[M]
}

/// A composite of Lipschitz maps is Lipschitz with the product of the constants.
theorem lipschitz_with_compose[M: MetricSpace, N: MetricSpace, P: MetricSpace](kf: Real, kg: Real, f: N -> P, g: M -> N) {
    is_lipschitz_with(kf, f) and is_lipschitz_with(kg, g)
    implies is_lipschitz_with(kf * kg, compose(f, g))
} by {
    if is_lipschitz_with(kf, f) and is_lipschitz_with(kg, g) {
        not kg.is_negative
        forall(x: M, y: M) {
            distance_non_negative(x, y)
            lipschitz_with_distance(kg, g, x, y)
            lipschitz_with_distance(kf, f, g(x), g(y))
            kf * g(x).distance(g(y)) <= kf * (kg * x.distance(y))
            lte_trans(
                f(g(x)).distance(f(g(y))),
                kf * g(x).distance(g(y)),
                (kf * kg) * x.distance(y)
            )
            f(g(x)).distance(f(g(y))) <= (kf * kg) * x.distance(y)
            compose(f, g)(x).distance(compose(f, g)(y)) <= (kf * kg) * x.distance(y)
        }
        lipschitz_bound_from_distances(kf * kg, compose(f, g))
        lipschitz_bound(kf * kg, compose(f, g))
        is_lipschitz_with(kf * kg, compose(f, g))
    }
}

/// A composite of Lipschitz maps is Lipschitz.
theorem lipschitz_compose[M: MetricSpace, N: MetricSpace, P: MetricSpace](f: N -> P, g: M -> N) {
    is_lipschitz(f) and is_lipschitz(g) implies is_lipschitz(compose(f, g))
} by {
    if is_lipschitz(f) and is_lipschitz(g) {
        let kf: Real satisfy {
            is_lipschitz_with(kf, f)
        }
        let kg: Real satisfy {
            is_lipschitz_with(kg, g)
        }
        lipschitz_with_compose(kf, kg, f, g)
        is_lipschitz(compose(f, g))
    }
}

/// True if a map preserves all metric distances.
define is_isometry[M: MetricSpace, N: MetricSpace](f: M -> N) -> Bool {
    forall(x: M, y: M) {
        f(x).distance(f(y)) = x.distance(y)
    }
}

/// An isometry preserves the distance between a specific pair of points.
theorem isometry_distance[M: MetricSpace, N: MetricSpace](f: M -> N, x: M, y: M) {
    is_isometry(f) implies f(x).distance(f(y)) = x.distance(y)
} by {
    if is_isometry(f) {
        is_isometry(f) = forall(a: M, b: M) {
            f(a).distance(f(b)) = a.distance(b)
        }
        f(x).distance(f(y)) = x.distance(y)
    }
}

/// An isometry is injective.
theorem isometry_is_injective_fn[M: MetricSpace, N: MetricSpace](f: M -> N) {
    is_isometry(f) implies is_injective_fn(f)
} by {
    if is_isometry(f) {
        forall(x: M, y: M) {
            if f(x) = f(y) {
                isometry_distance(f, x, y)
                f(x).distance(f(y)) = x.distance(y)
                x.distance(y) = 0
                x = y
            }
        }
        is_injective_fn(f)
    }
}

/// The identity map is an isometry.
theorem identity_is_isometry[M: MetricSpace] {
    is_isometry(identity_fn[M])
} by {
    forall(x: M, y: M) {
        identity_fn[M](x).distance(identity_fn[M](y)) = x.distance(y)
    }
}

/// A composite of isometries is an isometry.
theorem isometry_compose[M: MetricSpace, N: MetricSpace, P: MetricSpace](f: N -> P, g: M -> N) {
    is_isometry(f) and is_isometry(g) implies is_isometry(compose(f, g))
} by {
    if is_isometry(f) and is_isometry(g) {
        forall(x: M, y: M) {
            isometry_distance(f, g(x), g(y))
            isometry_distance(g, x, y)
            compose(f, g)(x).distance(compose(f, g)(y)) = x.distance(y)
        }
        is_isometry(compose(f, g))
    }
}

/// A composite of isometries is injective.
theorem isometry_compose_is_injective_fn[M: MetricSpace, N: MetricSpace, P: MetricSpace](f: N -> P, g: M -> N) {
    is_isometry(f) and is_isometry(g) implies is_injective_fn(compose(f, g))
} by {
    if is_isometry(f) and is_isometry(g) {
        isometry_is_injective_fn(f)
        isometry_is_injective_fn(g)
        compose_injective_fn(f, g)
        is_injective_fn(compose(f, g))
    }
}

/// An isometry is Lipschitz with constant one.
theorem isometry_is_lipschitz_with_one[M: MetricSpace, N: MetricSpace](f: M -> N) {
    is_isometry(f) implies is_lipschitz_with(1, f)
} by {
    if is_isometry(f) {
        not Real.1.is_negative
        forall(x: M, y: M) {
            isometry_distance(f, x, y)
            f(x).distance(f(y)) = x.distance(y)
            f(x).distance(f(y)) <= Real.1 * x.distance(y)
        }
        lipschitz_bound_from_distances[M, N](Real.1, f)
        is_lipschitz_with(1, f)
    }
}

/// An isometry is Lipschitz.
theorem isometry_is_lipschitz[M: MetricSpace, N: MetricSpace](f: M -> N) {
    is_isometry(f) implies is_lipschitz(f)
} by {
    if is_isometry(f) {
        isometry_is_lipschitz_with_one(f)
        is_lipschitz(f)
    }
}

/// A Lipschitz map sends a source closed-ball containment to an explicit image closed-ball containment.
theorem lipschitz_with_closed_ball_image_subset[M: MetricSpace, N: MetricSpace](
    k: Real, f: M -> N, s: Set[M], c: M, r: Real
) {
    is_lipschitz_with(k, f) and s.subset(closed_ball(c, r)) and not r.is_negative implies
    set_image(s, f).subset(closed_ball(f(c), k * r))
} by {
    if is_lipschitz_with(k, f) and s.subset(closed_ball(c, r)) and not r.is_negative {
        not k.is_negative
        forall(y: N) {
            if set_image(s, f).contains(y) {
                set_image_contains_witness(s, f, y)
                let x: M satisfy {
                    s.contains(x) and y = f(x)
                }
                subset_contains(s, closed_ball(c, r), x)
                lipschitz_with_distance(k, f, x, c)
                lte_trans(f(x).distance(f(c)), k * x.distance(c), k * r)
                f(x).distance(f(c)) <= k * r
                closed_ball(f(c), k * r).contains(y)
            }
        }
    }
}

/// A Lipschitz map sends every nonempty bounded set to a bounded image set.
theorem lipschitz_with_bounded_nonempty_image_is_bounded[M: MetricSpace, N: MetricSpace](
    k: Real, f: M -> N, s: Set[M]
) {
    is_lipschitz_with(k, f) and is_bounded(s) and exists(x: M) { s.contains(x) } implies
    is_bounded(set_image(s, f))
} by {
    if is_lipschitz_with(k, f) and is_bounded(s) and exists(x: M) { s.contains(x) } {
        let c: M satisfy {
            exists(r: Real) { s.subset(closed_ball(c, r)) }
        }
        let r: Real satisfy {
            s.subset(closed_ball(c, r))
        }
        let x: M satisfy {
            s.contains(x)
        }
        subset_contains(s, closed_ball(c, r), x)
        distance_non_negative(x, c)
        Real.0 <= x.distance(c)
        lte_trans(Real.0, x.distance(c), r)
        Real.0 <= r
        lipschitz_with_closed_ball_image_subset(k, f, s, c, r)
        exists(center: N, radius: Real) {
            set_image(s, f).subset(closed_ball(center, radius))
        }
        is_bounded(set_image(s, f))
    }
}

/// A Lipschitz map sends every nonempty bounded set to a bounded image set.
theorem lipschitz_bounded_nonempty_image_is_bounded[M: MetricSpace, N: MetricSpace](f: M -> N, s: Set[M]) {
    is_lipschitz(f) and is_bounded(s) and exists(x: M) { s.contains(x) } implies
    is_bounded(set_image(s, f))
} by {
    if is_lipschitz(f) and is_bounded(s) and exists(x: M) { s.contains(x) } {
        let k: Real satisfy {
            is_lipschitz_with(k, f)
        }
        lipschitz_with_bounded_nonempty_image_is_bounded(k, f, s)
        is_bounded(set_image(s, f))
    }
}

/// An isometry sends every nonempty bounded set to a bounded image set.
theorem isometry_bounded_nonempty_image_is_bounded[M: MetricSpace, N: MetricSpace](f: M -> N, s: Set[M]) {
    is_isometry(f) and is_bounded(s) and exists(x: M) { s.contains(x) } implies
    is_bounded(set_image(s, f))
} by {
    if is_isometry(f) and is_bounded(s) and exists(x: M) { s.contains(x) } {
        isometry_is_lipschitz_with_one(f)
        lipschitz_with_bounded_nonempty_image_is_bounded(1, f, s)
        is_bounded(set_image(s, f))
    }
}

/// An isometry carries open-ball membership exactly to open-ball membership at the image center.
theorem isometry_open_ball_contains_iff[M: MetricSpace, N: MetricSpace](f: M -> N, x: M, r: Real, y: M) {
    is_isometry(f) implies open_ball(f(x), r).contains(f(y)) = open_ball(x, r).contains(y)
} by {
    if is_isometry(f) {
        isometry_distance(f, y, x)
        f(y).distance(f(x)) = y.distance(x)
        open_ball(f(x), r).contains(f(y)) = open_ball(x, r).contains(y)
    }
}

/// An isometry carries closed-ball membership exactly to closed-ball membership at the image center.
theorem isometry_closed_ball_contains_iff[M: MetricSpace, N: MetricSpace](f: M -> N, x: M, r: Real, y: M) {
    is_isometry(f) implies closed_ball(f(x), r).contains(f(y)) = closed_ball(x, r).contains(y)
} by {
    if is_isometry(f) {
        isometry_distance(f, y, x)
        f(y).distance(f(x)) = y.distance(x)
        closed_ball(f(x), r).contains(f(y)) = closed_ball(x, r).contains(y)
    }
}

/// An isometry carries membership in an open ball to membership in the corresponding image open ball.
theorem isometry_open_ball_forward[M: MetricSpace, N: MetricSpace](f: M -> N, x: M, r: Real, y: M) {
    is_isometry(f) and open_ball(x, r).contains(y) implies open_ball(f(x), r).contains(f(y))
} by {
    if is_isometry(f) and open_ball(x, r).contains(y) {
        isometry_open_ball_contains_iff(f, x, r, y)
        open_ball(f(x), r).contains(f(y))
    }
}

/// An isometry reflects membership in an image open ball.
theorem isometry_open_ball_backward[M: MetricSpace, N: MetricSpace](f: M -> N, x: M, r: Real, y: M) {
    is_isometry(f) and open_ball(f(x), r).contains(f(y)) implies open_ball(x, r).contains(y)
} by {
    if is_isometry(f) and open_ball(f(x), r).contains(f(y)) {
        isometry_open_ball_contains_iff(f, x, r, y)
        open_ball(x, r).contains(y)
    }
}

/// An isometry carries membership in a closed ball to membership in the corresponding image closed ball.
theorem isometry_closed_ball_forward[M: MetricSpace, N: MetricSpace](f: M -> N, x: M, r: Real, y: M) {
    is_isometry(f) and closed_ball(x, r).contains(y) implies closed_ball(f(x), r).contains(f(y))
} by {
    if is_isometry(f) and closed_ball(x, r).contains(y) {
        isometry_closed_ball_contains_iff(f, x, r, y)
        closed_ball(f(x), r).contains(f(y))
    }
}

/// An isometry reflects membership in an image closed ball.
theorem isometry_closed_ball_backward[M: MetricSpace, N: MetricSpace](f: M -> N, x: M, r: Real, y: M) {
    is_isometry(f) and closed_ball(f(x), r).contains(f(y)) implies closed_ball(x, r).contains(y)
} by {
    if is_isometry(f) and closed_ball(f(x), r).contains(f(y)) {
        isometry_closed_ball_contains_iff(f, x, r, y)
        closed_ball(x, r).contains(y)
    }
}

/// The image of an open ball under an isometry lies in the corresponding open ball.
theorem isometry_open_ball_image_subset[M: MetricSpace, N: MetricSpace](f: M -> N, x: M, r: Real) {
    is_isometry(f) implies set_image(open_ball(x, r), f).subset(open_ball(f(x), r))
} by {
    if is_isometry(f) {
        forall(z: N) {
            if set_image(open_ball(x, r), f).contains(z) {
                set_image_contains_witness(open_ball(x, r), f, z)
                let y: M satisfy {
                    open_ball(x, r).contains(y) and z = f(y)
                }
                isometry_open_ball_forward(f, x, r, y)
                open_ball(f(x), r).contains(z)
            }
        }
    }
}

/// The image of a closed ball under an isometry lies in the corresponding closed ball.
theorem isometry_closed_ball_image_subset[M: MetricSpace, N: MetricSpace](f: M -> N, x: M, r: Real) {
    is_isometry(f) implies set_image(closed_ball(x, r), f).subset(closed_ball(f(x), r))
} by {
    if is_isometry(f) {
        forall(z: N) {
            if set_image(closed_ball(x, r), f).contains(z) {
                set_image_contains_witness(closed_ball(x, r), f, z)
                let y: M satisfy {
                    closed_ball(x, r).contains(y) and z = f(y)
                }
                isometry_closed_ball_forward(f, x, r, y)
                closed_ball(f(x), r).contains(z)
            }
        }
    }
}

/// An isometry preserves the tail-bound form of metric convergence.
theorem isometry_preserves_tendsto_metric_bounds[M: MetricSpace, N: MetricSpace](f: M -> N, q: Nat -> M, a: M) {
    is_isometry(f) and tendsto_metric(q, a) implies forall(eps: Real) {
        eps.is_positive implies exists(n: Nat) {
            metric_tail_bound(compose(f, q), f(a), n, eps)
        }
    }
} by {
    if is_isometry(f) and tendsto_metric(q, a) {
        forall(eps: Real) {
            if eps.is_positive {
                tendsto_metric_tail_bound(q, a, eps)
                let n: Nat satisfy {
                    metric_tail_bound(q, a, n, eps)
                }
                forall(i: Nat) {
                    if n <= i {
                        metric_tail_bound_at(q, a, n, eps, i)
                        q(i).distance(a) < eps
                        isometry_distance(f, q(i), a)
                        compose(f, q)(i).distance(f(a)) < eps
                    }
                }
                metric_tail_bound(compose(f, q), f(a), n, eps)
                exists(m: Nat) { metric_tail_bound(compose(f, q), f(a), m, eps) }
            }
        }
    }
}

/// True if every point of `s` has some open ball around it contained in `s`.
define is_open_metric[M: MetricSpace](s: Set[M]) -> Bool {
    forall(x: M) {
        s.contains(x) implies exists(r: Real) {
            r.is_positive and open_ball(x, r).subset(s)
        }
    }
}

/// True if the complement of `s` is open.
define is_closed_metric[M: MetricSpace](s: Set[M]) -> Bool {
    is_open_metric(s.c)
}

/// The empty set is open.
theorem empty_set_is_open_metric[M: MetricSpace] {
    is_open_metric(Set[M].empty_set)
} by {
    forall(x: M) {
        if (Set[M].empty_set).contains(x) {
            false
        }
    }
}

/// The universal set is open.
theorem universal_set_is_open_metric[M: MetricSpace] {
    is_open_metric(Set[M].universal_set)
} by {
    forall(x: M) {
        if (Set[M].universal_set).contains(x) {
            forall(y: M) {
                if open_ball(x, 1).contains(y) {
                    (Set[M].universal_set).contains(y)
                }
            }
            Real.1.is_positive
        }
    }
}

/// Inside an open ball, a smaller ball around any point fits inside.
theorem open_ball_inner_ball[M: MetricSpace](x: M, r: Real, y: M) {
    open_ball(x, r).contains(y) implies exists(s: Real) {
        s.is_positive and open_ball(y, s).subset(open_ball(x, r))
    }
} by {
    if open_ball(x, r).contains(y) {
        y.distance(x) < r
        lt_imp_minus_pos(y.distance(x), r)
        (r - y.distance(x)).is_positive
        eps_lt_half(r - y.distance(x))
        let s: Real satisfy {
            s.is_positive and s + s < r - y.distance(x)
        }
        s + s < r - y.distance(x)
        s < s + s
        s < r - y.distance(x)
        s + y.distance(x) < (r - y.distance(x)) + y.distance(x)
        (r - y.distance(x)) + y.distance(x) = r
        s + y.distance(x) < r
        forall(z: M) {
            if open_ball(y, s).contains(z) {
                z.distance(y) < s
                z.distance(y) + y.distance(x) < s + y.distance(x)
                z.distance(y) + y.distance(x) < r
                lte_lt_trans(z.distance(x), z.distance(y) + y.distance(x), r)
                z.distance(x) < r
                open_ball(x, r).contains(z)
            }
        }
        open_ball(y, s).subset(open_ball(x, r))
        exists(t: Real) {
            t.is_positive and open_ball(y, t).subset(open_ball(x, r))
        }
    }
}

/// A set is open when every point has a positive radius open ball inside it.
theorem is_open_metric_from_inner_balls[M: MetricSpace](s: Set[M]) {
    (forall(x: M) {
        s.contains(x) implies exists(r: Real) {
            r.is_positive and open_ball(x, r).subset(s)
        }
    }) implies is_open_metric(s)
} by {
    if forall(x: M) {
        s.contains(x) implies exists(r: Real) {
            r.is_positive and open_ball(x, r).subset(s)
        }
    } {
        is_open_metric(s) = forall(x: M) {
            s.contains(x) implies exists(r: Real) {
                r.is_positive and open_ball(x, r).subset(s)
            }
        }
        is_open_metric(s)
    }
}

/// Every open ball is open.
theorem open_ball_is_open_metric[M: MetricSpace](xc: M, rad: Real) {
    is_open_metric(open_ball(xc, rad))
} by {
    let b: Set[M] = open_ball(xc, rad)
    forall(y: M) {
        if b.contains(y) {
            open_ball_inner_ball(xc, rad, y)
            let s: Real satisfy {
                s.is_positive and open_ball(y, s).subset(open_ball(xc, rad))
            }
            exists(rr: Real) {
                rr.is_positive and open_ball(y, rr).subset(b)
            }
        }
    }
    is_open_metric_from_inner_balls(b)
    is_open_metric(b)
}

/// The union of two open sets is open.
theorem union_of_open_metric_is_open[M: MetricSpace](s: Set[M], t: Set[M]) {
    is_open_metric(s) and is_open_metric(t) implies is_open_metric(s.union(t))
} by {
    if is_open_metric(s) and is_open_metric(t) {
        is_open_metric(s) = forall(a: M) {
            s.contains(a) implies exists(r: Real) {
                r.is_positive and open_ball(a, r).subset(s)
            }
        }
        is_open_metric(t) = forall(a: M) {
            t.contains(a) implies exists(r: Real) {
                r.is_positive and open_ball(a, r).subset(t)
            }
        }
        forall(x: M) {
            if s.union(t).contains(x) {
                s.union(t).contains(x) = (s.contains(x) or t.contains(x))
                s.contains(x) or t.contains(x)
                if s.contains(x) {
                    let r: Real satisfy {
                        r.is_positive and open_ball(x, r).subset(s)
                    }
                    forall(y: M) {
                        if open_ball(x, r).contains(y) {
                            s.contains(y)
                            s.union(t).contains(y)
                        }
                    }
                    open_ball(x, r).subset(s.union(t))
                    exists(rr: Real) {
                        rr.is_positive and open_ball(x, rr).subset(s.union(t))
                    }
                } else {
                    t.contains(x)
                    let r: Real satisfy {
                        r.is_positive and open_ball(x, r).subset(t)
                    }
                    forall(y: M) {
                        if open_ball(x, r).contains(y) {
                            t.contains(y)
                            s.union(t).contains(y)
                        }
                    }
                    open_ball(x, r).subset(s.union(t))
                    exists(rr: Real) {
                        rr.is_positive and open_ball(x, rr).subset(s.union(t))
                    }
                }
            }
        }
    }
}

/// The intersection of two open sets is open.
theorem intersection_of_open_metric_is_open[M: MetricSpace](s: Set[M], t: Set[M]) {
    is_open_metric(s) and is_open_metric(t) implies is_open_metric(s.intersection(t))
} by {
    if is_open_metric(s) and is_open_metric(t) {
        is_open_metric(s) = forall(a: M) {
            s.contains(a) implies exists(r: Real) {
                r.is_positive and open_ball(a, r).subset(s)
            }
        }
        is_open_metric(t) = forall(a: M) {
            t.contains(a) implies exists(r: Real) {
                r.is_positive and open_ball(a, r).subset(t)
            }
        }
        forall(x: M) {
            if s.intersection(t).contains(x) {
                s.intersection(t).contains(x) = (s.contains(x) and t.contains(x))
                s.contains(x)
                t.contains(x)
                let rs: Real satisfy {
                    rs.is_positive and open_ball(x, rs).subset(s)
                }
                let rt: Real satisfy {
                    rt.is_positive and open_ball(x, rt).subset(t)
                }
                let r: Real = if rs <= rt { rs } else { rt }
                r.is_positive
                r <= rs
                r <= rt
                forall(y: M) {
                    if open_ball(x, r).contains(y) {
                        y.distance(x) < r
                        y.distance(x) < rs
                        y.distance(x) < rt
                        open_ball(x, rs).contains(y)
                        open_ball(x, rt).contains(y)
                        s.contains(y)
                        t.contains(y)
                        s.intersection(t).contains(y)
                    }
                }
                open_ball(x, r).subset(s.intersection(t))
                exists(rr: Real) {
                    rr.is_positive and open_ball(x, rr).subset(s.intersection(t))
                }
            }
        }
    }
}

/// The empty set is closed.
theorem empty_set_is_closed_metric[M: MetricSpace] {
    is_closed_metric(Set[M].empty_set)
} by {
    universal_set_is_open_metric[M]
}

/// The universal set is closed.
theorem universal_set_is_closed_metric[M: MetricSpace] {
    is_closed_metric(Set[M].universal_set)
} by {
    empty_set_is_open_metric[M]
}

/// If `y` is outside the closed ball at `xc` of radius `rad`, then any point within
/// distance `y.distance(xc) - rad` of `y` is also outside the closed ball.
theorem closed_ball_complement_inner[M: MetricSpace](xc: M, rad: Real, y: M, z: M) {
    rad < y.distance(xc) and z.distance(y) < y.distance(xc) - rad
        implies rad < z.distance(xc)
} by {
    if rad < y.distance(xc) and z.distance(y) < y.distance(xc) - rad {
        // Step 1: z.distance(y) + rad < y.distance(xc).
        y.distance(xc) - rad + rad = y.distance(xc)
        z.distance(y) + rad < (y.distance(xc) - rad) + rad
        // Step 2: y.distance(xc) <= z.distance(y) + z.distance(xc) by triangle.
        y.distance(xc) <= z.distance(y) + z.distance(xc)
        // Combine: z.distance(y) + rad < z.distance(y) + z.distance(xc), then cancel.
        lt_lte_trans(z.distance(y) + rad, y.distance(xc), z.distance(y) + z.distance(xc))
        z.distance(y) + rad < z.distance(y) + z.distance(xc)
        rad + z.distance(y) < z.distance(xc) + z.distance(y)
        lt_add_converse(rad, z.distance(xc), z.distance(y))
        rad < z.distance(xc)
    }
}

/// If `y` is outside the closed ball at `xc` of radius `rad`, then the open ball
/// around `y` of radius `y.distance(xc) - rad` lies in the complement.
theorem closed_ball_complement_open_ball_subset[M: MetricSpace](xc: M, rad: Real, y: M) {
    rad < y.distance(xc) implies
        open_ball(y, y.distance(xc) - rad).subset(closed_ball(xc, rad).c)
} by {
    if rad < y.distance(xc) {
        forall(z: M) {
            if open_ball(y, y.distance(xc) - rad).contains(z) {
                closed_ball_complement_inner(xc, rad, y, z)
                rad < z.distance(xc)
                not (z.distance(xc) <= rad)
                closed_ball(xc, rad).c.contains(z)
            }
        }
    }
}

/// Each point outside a closed ball has a positive-radius open ball around it
/// contained in the complement.
theorem closed_ball_complement_has_open_neighborhood[M: MetricSpace](xc: M, rad: Real, y: M) {
    closed_ball(xc, rad).c.contains(y) implies exists(r: Real) {
        r.is_positive and open_ball(y, r).subset(closed_ball(xc, rad).c)
    }
} by {
    if closed_ball(xc, rad).c.contains(y) {
        not closed_ball(xc, rad).contains(y)
        not (y.distance(xc) <= rad)
        rad < y.distance(xc)
        lt_imp_minus_pos(rad, y.distance(xc))
        closed_ball_complement_open_ball_subset(xc, rad, y)
        (y.distance(xc) - rad).is_positive and open_ball(y, y.distance(xc) - rad).subset(closed_ball(xc, rad).c)
    }
}

/// An isometry preserves the tail-bound form of metric Cauchy sequences.
theorem isometry_preserves_cauchy_metric_bounds[M: MetricSpace, N: MetricSpace](f: M -> N, q: Nat -> M) {
    is_isometry(f) and is_cauchy_metric(q) implies forall(eps: Real) {
        eps.is_positive implies exists(n: Nat) {
            cauchy_metric_bound(compose(f, q), n, eps)
        }
    }
} by {
    if is_isometry(f) and is_cauchy_metric(q) {
        forall(eps: Real) {
            if eps.is_positive {
                let n: Nat satisfy {
                    cauchy_metric_bound(q, n, eps)
                }
                forall(i: Nat, j: Nat) {
                    if n <= i and n <= j {
                        cauchy_metric_bound_at(q, n, eps, i, j)
                        q(i).distance(q(j)) < eps
                        isometry_distance(f, q(i), q(j))
                        compose(f, q)(i).distance(compose(f, q)(j)) < eps
                    }
                }
                cauchy_metric_bound(compose(f, q), n, eps)
                exists(m: Nat) { cauchy_metric_bound(compose(f, q), m, eps) }
            }
        }
    }
}

/// An isometry preserves metric convergence.
theorem isometry_preserves_tendsto_metric[M: MetricSpace, N: MetricSpace](
    f: M -> N, q: Nat -> M, a: M
) {
    is_isometry(f) and tendsto_metric(q, a) implies tendsto_metric(compose(f, q), f(a))
} by {
    if is_isometry(f) and tendsto_metric(q, a) {
        isometry_preserves_tendsto_metric_bounds(f, q, a)
        tendsto_metric_from_tail_bounds(compose(f, q), f(a))
        tendsto_metric(compose(f, q), f(a))
    }
}

/// An isometry preserves metric Cauchy sequences.
theorem isometry_preserves_cauchy_metric[M: MetricSpace, N: MetricSpace](
    f: M -> N, q: Nat -> M
) {
    is_isometry(f) and is_cauchy_metric(q) implies is_cauchy_metric(compose(f, q))
} by {
    if is_isometry(f) and is_cauchy_metric(q) {
        isometry_preserves_cauchy_metric_bounds(f, q)
        is_cauchy_metric_from_bounds(compose(f, q))
        is_cauchy_metric(compose(f, q))
    }
}

/// A one-Lipschitz map preserves a concrete metric tail bound.
theorem lipschitz_with_one_preserves_metric_tail_bound[M: MetricSpace, N: MetricSpace](
    f: M -> N, q: Nat -> M, a: M, n: Nat, eps: Real
) {
    is_lipschitz_with(1, f) and metric_tail_bound(q, a, n, eps)
        implies metric_tail_bound(compose(f, q), f(a), n, eps)
} by {
    if is_lipschitz_with(1, f) and metric_tail_bound(q, a, n, eps) {
        forall(i: Nat) {
            if n <= i {
                metric_tail_bound_at(q, a, n, eps, i)
                lipschitz_with_distance(1, f, q(i), a)
                f(q(i)).distance(f(a)) <= q(i).distance(a)
                lte_lt_trans(f(q(i)).distance(f(a)), q(i).distance(a), eps)
                f(q(i)).distance(f(a)) < eps
                compose(f, q)(i).distance(f(a)) < eps
            }
        }
        metric_tail_bound_from_distances(compose(f, q), f(a), n, eps)
        metric_tail_bound(compose(f, q), f(a), n, eps)
    }
}

/// A one-Lipschitz map preserves the tail-bound form of metric convergence.
theorem lipschitz_with_one_preserves_tendsto_metric_bounds[M: MetricSpace, N: MetricSpace](
    f: M -> N, q: Nat -> M, a: M
) {
    is_lipschitz_with(1, f) and tendsto_metric(q, a) implies forall(eps: Real) {
        eps.is_positive implies exists(n: Nat) {
            metric_tail_bound(compose(f, q), f(a), n, eps)
        }
    }
} by {
    if is_lipschitz_with(1, f) and tendsto_metric(q, a) {
        forall(eps: Real) {
            if eps.is_positive {
                tendsto_metric_tail_bound(q, a, eps)
                let n: Nat satisfy {
                    metric_tail_bound(q, a, n, eps)
                }
                lipschitz_with_one_preserves_metric_tail_bound(f, q, a, n, eps)
                exists(m: Nat) { metric_tail_bound(compose(f, q), f(a), m, eps) }
            }
        }
    }
}

/// A one-Lipschitz map preserves metric convergence.
theorem lipschitz_with_one_preserves_tendsto_metric[M: MetricSpace, N: MetricSpace](
    f: M -> N, q: Nat -> M, a: M
) {
    is_lipschitz_with(1, f) and tendsto_metric(q, a) implies tendsto_metric(compose(f, q), f(a))
} by {
    if is_lipschitz_with(1, f) and tendsto_metric(q, a) {
        lipschitz_with_one_preserves_tendsto_metric_bounds(f, q, a)
        tendsto_metric_from_tail_bounds(compose(f, q), f(a))
        tendsto_metric(compose(f, q), f(a))
    }
}

/// A one-Lipschitz map preserves a concrete Cauchy metric bound.
theorem lipschitz_with_one_preserves_cauchy_metric_bound[M: MetricSpace, N: MetricSpace](
    f: M -> N, q: Nat -> M, n: Nat, eps: Real
) {
    is_lipschitz_with(1, f) and cauchy_metric_bound(q, n, eps)
        implies cauchy_metric_bound(compose(f, q), n, eps)
} by {
    if is_lipschitz_with(1, f) and cauchy_metric_bound(q, n, eps) {
        let p = compose(f, q)
        forall(i: Nat, j: Nat) {
            if n <= i and n <= j {
                cauchy_metric_bound_at(q, n, eps, i, j)
                lipschitz_with_distance(1, f, q(i), q(j))
                f(q(i)).distance(f(q(j))) <= q(i).distance(q(j))
                lte_lt_trans(f(q(i)).distance(f(q(j))), q(i).distance(q(j)), eps)
                f(q(i)).distance(f(q(j))) < eps
                p(i).distance(p(j)) < eps
            }
        }
        if not cauchy_metric_bound(p, n, eps) {
            let i: Nat satisfy {
                exists(j: Nat) {
                    n <= i and n <= j and not p(i).distance(p(j)) < eps
                }
            }
            let j: Nat satisfy {
                n <= i and n <= j and not p(i).distance(p(j)) < eps
            }
            false
        }
        cauchy_metric_bound(compose(f, q), n, eps)
    }
}

/// A one-Lipschitz map preserves the tail-bound form of metric Cauchy sequences.
theorem lipschitz_with_one_preserves_cauchy_metric_bounds[M: MetricSpace, N: MetricSpace](
    f: M -> N, q: Nat -> M
) {
    is_lipschitz_with(1, f) and is_cauchy_metric(q) implies forall(eps: Real) {
        eps.is_positive implies exists(n: Nat) {
            cauchy_metric_bound(compose(f, q), n, eps)
        }
    }
} by {
    if is_lipschitz_with(1, f) and is_cauchy_metric(q) {
        forall(eps: Real) {
            if eps.is_positive {
                is_cauchy_metric_bound(q, eps)
                let n: Nat satisfy {
                    cauchy_metric_bound(q, n, eps)
                }
                lipschitz_with_one_preserves_cauchy_metric_bound(f, q, n, eps)
                exists(m: Nat) { cauchy_metric_bound(compose(f, q), m, eps) }
            }
        }
    }
}

/// A one-Lipschitz map preserves metric Cauchy sequences.
theorem lipschitz_with_one_preserves_cauchy_metric[M: MetricSpace, N: MetricSpace](
    f: M -> N, q: Nat -> M
) {
    is_lipschitz_with(1, f) and is_cauchy_metric(q) implies is_cauchy_metric(compose(f, q))
} by {
    if is_lipschitz_with(1, f) and is_cauchy_metric(q) {
        lipschitz_with_one_preserves_cauchy_metric_bounds(f, q)
        is_cauchy_metric_from_bounds(compose(f, q))
        is_cauchy_metric(compose(f, q))
    }
}

/// A Lipschitz map with constant at most one preserves a concrete metric tail bound.
theorem lipschitz_with_le_one_preserves_metric_tail_bound[M: MetricSpace, N: MetricSpace](
    k: Real, f: M -> N, q: Nat -> M, a: M, n: Nat, eps: Real
) {
    is_lipschitz_with(k, f) and k <= 1 and metric_tail_bound(q, a, n, eps)
        implies metric_tail_bound(compose(f, q), f(a), n, eps)
} by {
    if is_lipschitz_with(k, f) and k <= 1 and metric_tail_bound(q, a, n, eps) {
        forall(i: Nat) {
            if n <= i {
                metric_tail_bound_at(q, a, n, eps, i)
                lipschitz_with_le_one_distance(k, f, q(i), a)
                f(q(i)).distance(f(a)) <= q(i).distance(a)
                lte_lt_trans(f(q(i)).distance(f(a)), q(i).distance(a), eps)
                f(q(i)).distance(f(a)) < eps
                compose(f, q)(i).distance(f(a)) < eps
            }
        }
        metric_tail_bound_from_distances(compose(f, q), f(a), n, eps)
        metric_tail_bound(compose(f, q), f(a), n, eps)
    }
}

/// A Lipschitz map with constant at most one preserves the tail-bound form of metric convergence.
theorem lipschitz_with_le_one_preserves_tendsto_metric_bounds[M: MetricSpace, N: MetricSpace](
    k: Real, f: M -> N, q: Nat -> M, a: M
) {
    is_lipschitz_with(k, f) and k <= 1 and tendsto_metric(q, a) implies forall(eps: Real) {
        eps.is_positive implies exists(n: Nat) {
            metric_tail_bound(compose(f, q), f(a), n, eps)
        }
    }
} by {
    if is_lipschitz_with(k, f) and k <= 1 and tendsto_metric(q, a) {
        forall(eps: Real) {
            if eps.is_positive {
                tendsto_metric_tail_bound(q, a, eps)
                let n: Nat satisfy {
                    metric_tail_bound(q, a, n, eps)
                }
                lipschitz_with_le_one_preserves_metric_tail_bound(k, f, q, a, n, eps)
                exists(m: Nat) { metric_tail_bound(compose(f, q), f(a), m, eps) }
            }
        }
    }
}

/// A Lipschitz map with constant at most one preserves metric convergence.
theorem lipschitz_with_le_one_preserves_tendsto_metric[M: MetricSpace, N: MetricSpace](
    k: Real, f: M -> N, q: Nat -> M, a: M
) {
    is_lipschitz_with(k, f) and k <= 1 and tendsto_metric(q, a) implies tendsto_metric(compose(f, q), f(a))
} by {
    if is_lipschitz_with(k, f) and k <= 1 and tendsto_metric(q, a) {
        lipschitz_with_le_one_preserves_tendsto_metric_bounds(k, f, q, a)
        tendsto_metric_from_tail_bounds(compose(f, q), f(a))
        tendsto_metric(compose(f, q), f(a))
    }
}

/// A Lipschitz map with constant at most one preserves a concrete Cauchy metric bound.
theorem lipschitz_with_le_one_preserves_cauchy_metric_bound[M: MetricSpace, N: MetricSpace](
    k: Real, f: M -> N, q: Nat -> M, n: Nat, eps: Real
) {
    is_lipschitz_with(k, f) and k <= 1 and cauchy_metric_bound(q, n, eps)
        implies cauchy_metric_bound(compose(f, q), n, eps)
} by {
    if is_lipschitz_with(k, f) and k <= 1 and cauchy_metric_bound(q, n, eps) {
        let p = compose(f, q)
        forall(i: Nat, j: Nat) {
            if n <= i and n <= j {
                cauchy_metric_bound_at(q, n, eps, i, j)
                lipschitz_with_le_one_distance(k, f, q(i), q(j))
                f(q(i)).distance(f(q(j))) <= q(i).distance(q(j))
                lte_lt_trans(f(q(i)).distance(f(q(j))), q(i).distance(q(j)), eps)
                f(q(i)).distance(f(q(j))) < eps
                p(i).distance(p(j)) < eps
            }
        }
        if not cauchy_metric_bound(p, n, eps) {
            let i: Nat satisfy {
                exists(j: Nat) {
                    n <= i and n <= j and not p(i).distance(p(j)) < eps
                }
            }
            let j: Nat satisfy {
                n <= i and n <= j and not p(i).distance(p(j)) < eps
            }
            false
        }
        cauchy_metric_bound(compose(f, q), n, eps)
    }
}

/// A Lipschitz map with constant at most one preserves the tail-bound form of metric Cauchy sequences.
theorem lipschitz_with_le_one_preserves_cauchy_metric_bounds[M: MetricSpace, N: MetricSpace](
    k: Real, f: M -> N, q: Nat -> M
) {
    is_lipschitz_with(k, f) and k <= 1 and is_cauchy_metric(q) implies forall(eps: Real) {
        eps.is_positive implies exists(n: Nat) {
            cauchy_metric_bound(compose(f, q), n, eps)
        }
    }
} by {
    if is_lipschitz_with(k, f) and k <= 1 and is_cauchy_metric(q) {
        forall(eps: Real) {
            if eps.is_positive {
                is_cauchy_metric_bound(q, eps)
                let n: Nat satisfy {
                    cauchy_metric_bound(q, n, eps)
                }
                lipschitz_with_le_one_preserves_cauchy_metric_bound(k, f, q, n, eps)
                exists(m: Nat) { cauchy_metric_bound(compose(f, q), m, eps) }
            }
        }
    }
}

/// A Lipschitz map with constant at most one preserves metric Cauchy sequences.
theorem lipschitz_with_le_one_preserves_cauchy_metric[M: MetricSpace, N: MetricSpace](
    k: Real, f: M -> N, q: Nat -> M
) {
    is_lipschitz_with(k, f) and k <= 1 and is_cauchy_metric(q) implies is_cauchy_metric(compose(f, q))
} by {
    if is_lipschitz_with(k, f) and k <= 1 and is_cauchy_metric(q) {
        lipschitz_with_le_one_preserves_cauchy_metric_bounds(k, f, q)
        is_cauchy_metric_from_bounds(compose(f, q))
        is_cauchy_metric(compose(f, q))
    }
}

/// A Lipschitz map turns a tail bound at `delta` into a tail bound at `eps` when `k * delta < eps`.
theorem lipschitz_with_preserves_metric_tail_bound_of_mul_lt[M: MetricSpace, N: MetricSpace](
    k: Real, f: M -> N, q: Nat -> M, a: M, n: Nat, delta: Real, eps: Real
) {
    is_lipschitz_with(k, f) and k * delta < eps and metric_tail_bound(q, a, n, delta)
        implies metric_tail_bound(compose(f, q), f(a), n, eps)
} by {
    if is_lipschitz_with(k, f) and k * delta < eps and metric_tail_bound(q, a, n, delta) {
        lipschitz_with_constant_non_negative(k, f)
        not k.is_negative
        forall(i: Nat) {
            if n <= i {
                metric_tail_bound_at(q, a, n, delta, i)
                q(i).distance(a) < delta
                q(i).distance(a) <= delta
                lipschitz_with_distance(k, f, q(i), a)
                lte_mul_nonneg_left(q(i).distance(a), delta, k)
                lte_trans(f(q(i)).distance(f(a)), k * q(i).distance(a), k * delta)
                lte_lt_trans(f(q(i)).distance(f(a)), k * delta, eps)
                f(q(i)).distance(f(a)) < eps
                compose(f, q)(i).distance(f(a)) < eps
            }
        }
        metric_tail_bound_from_distances(compose(f, q), f(a), n, eps)
        metric_tail_bound(compose(f, q), f(a), n, eps)
    }
}

/// A Lipschitz map preserves the tail-bound form of metric convergence.
theorem lipschitz_with_preserves_tendsto_metric_bounds[M: MetricSpace, N: MetricSpace](
    k: Real, f: M -> N, q: Nat -> M, a: M
) {
    is_lipschitz_with(k, f) and tendsto_metric(q, a) implies forall(eps: Real) {
        eps.is_positive implies exists(n: Nat) {
            metric_tail_bound(compose(f, q), f(a), n, eps)
        }
    }
} by {
    if is_lipschitz_with(k, f) and tendsto_metric(q, a) {
        lipschitz_with_constant_non_negative(k, f)
        forall(eps: Real) {
            if eps.is_positive {
                exists_small_mul(k, eps)
                let delta: Real satisfy {
                    delta.is_positive and k * delta < eps
                }
                tendsto_metric_tail_bound(q, a, delta)
                let n: Nat satisfy {
                    metric_tail_bound(q, a, n, delta)
                }
                lipschitz_with_preserves_metric_tail_bound_of_mul_lt(k, f, q, a, n, delta, eps)
                exists(m: Nat) { metric_tail_bound(compose(f, q), f(a), m, eps) }
            }
        }
    }
}

/// A Lipschitz map preserves metric convergence.
theorem lipschitz_with_preserves_tendsto_metric[M: MetricSpace, N: MetricSpace](
    k: Real, f: M -> N, q: Nat -> M, a: M
) {
    is_lipschitz_with(k, f) and tendsto_metric(q, a) implies tendsto_metric(compose(f, q), f(a))
} by {
    if is_lipschitz_with(k, f) and tendsto_metric(q, a) {
        lipschitz_with_preserves_tendsto_metric_bounds(k, f, q, a)
        tendsto_metric_from_tail_bounds(compose(f, q), f(a))
        tendsto_metric(compose(f, q), f(a))
    }
}

/// A Lipschitz map turns a Cauchy bound at `delta` into a Cauchy bound at `eps` when `k * delta < eps`.
theorem lipschitz_with_preserves_cauchy_metric_bound_of_mul_lt[M: MetricSpace, N: MetricSpace](
    k: Real, f: M -> N, q: Nat -> M, n: Nat, delta: Real, eps: Real
) {
    is_lipschitz_with(k, f) and k * delta < eps and cauchy_metric_bound(q, n, delta)
        implies cauchy_metric_bound(compose(f, q), n, eps)
} by {
    if is_lipschitz_with(k, f) and k * delta < eps and cauchy_metric_bound(q, n, delta) {
        let p = compose(f, q)
        lipschitz_with_constant_non_negative(k, f)
        not k.is_negative
        forall(i: Nat, j: Nat) {
            if n <= i and n <= j {
                cauchy_metric_bound_at(q, n, delta, i, j)
                q(i).distance(q(j)) < delta
                q(i).distance(q(j)) <= delta
                lipschitz_with_distance(k, f, q(i), q(j))
                lte_mul_nonneg_left(q(i).distance(q(j)), delta, k)
                lte_trans(f(q(i)).distance(f(q(j))), k * q(i).distance(q(j)), k * delta)
                lte_lt_trans(f(q(i)).distance(f(q(j))), k * delta, eps)
                f(q(i)).distance(f(q(j))) < eps
                p(i).distance(p(j)) < eps
            }
        }
        if not cauchy_metric_bound(p, n, eps) {
            let i: Nat satisfy {
                exists(j: Nat) {
                    n <= i and n <= j and not p(i).distance(p(j)) < eps
                }
            }
            let j: Nat satisfy {
                n <= i and n <= j and not p(i).distance(p(j)) < eps
            }
            false
        }
        cauchy_metric_bound(compose(f, q), n, eps)
    }
}

/// A Lipschitz map preserves the tail-bound form of metric Cauchy sequences.
theorem lipschitz_with_preserves_cauchy_metric_bounds[M: MetricSpace, N: MetricSpace](
    k: Real, f: M -> N, q: Nat -> M
) {
    is_lipschitz_with(k, f) and is_cauchy_metric(q) implies forall(eps: Real) {
        eps.is_positive implies exists(n: Nat) {
            cauchy_metric_bound(compose(f, q), n, eps)
        }
    }
} by {
    if is_lipschitz_with(k, f) and is_cauchy_metric(q) {
        lipschitz_with_constant_non_negative(k, f)
        forall(eps: Real) {
            if eps.is_positive {
                exists_small_mul(k, eps)
                let delta: Real satisfy {
                    delta.is_positive and k * delta < eps
                }
                is_cauchy_metric_bound(q, delta)
                let n: Nat satisfy {
                    cauchy_metric_bound(q, n, delta)
                }
                lipschitz_with_preserves_cauchy_metric_bound_of_mul_lt(k, f, q, n, delta, eps)
                exists(m: Nat) { cauchy_metric_bound(compose(f, q), m, eps) }
            }
        }
    }
}

/// A Lipschitz map preserves metric Cauchy sequences.
theorem lipschitz_with_preserves_cauchy_metric[M: MetricSpace, N: MetricSpace](
    k: Real, f: M -> N, q: Nat -> M
) {
    is_lipschitz_with(k, f) and is_cauchy_metric(q) implies is_cauchy_metric(compose(f, q))
} by {
    if is_lipschitz_with(k, f) and is_cauchy_metric(q) {
        lipschitz_with_preserves_cauchy_metric_bounds(k, f, q)
        is_cauchy_metric_from_bounds(compose(f, q))
        is_cauchy_metric(compose(f, q))
    }
}

/// The diameter of a set, defined as the supremum of pairwise distances when
/// the set is bounded and nonempty, and zero otherwise.
let diameter[M: MetricSpace](s: Set[M]) -> d: Real satisfy {
    if has_diameter(s) {
        is_diameter(s, d)
    } else {
        d = 0
    }
} by {
    if has_diameter(s) {
        has_diameter(s) = (is_bounded(s) and exists(x: M) { s.contains(x) })
        diameter_exists(s)
        let d0: Real satisfy { is_diameter(s, d0) }
        if has_diameter(s) {
            is_diameter(s, d0)
        } else {
            false
        }
        exists(d: Real) {
            if has_diameter(s) {
                is_diameter(s, d)
            } else {
                d = 0
            }
        }
    } else {
        let d0: Real = 0
        if has_diameter(s) {
            false
        } else {
            d0 = 0
        }
        exists(d: Real) {
            if has_diameter(s) {
                is_diameter(s, d)
            } else {
                d = 0
            }
        }
    }
}

/// When a set is bounded and nonempty, `diameter(s)` satisfies `is_diameter`.
theorem diameter_is_diameter[M: MetricSpace](s: Set[M]) {
    has_diameter(s) implies is_diameter(s, diameter(s))
}

/// When a set is not bounded or empty, `diameter(s)` is zero.
theorem diameter_default[M: MetricSpace](s: Set[M]) {
    not has_diameter(s) implies diameter(s) = 0
}

/// Any pairwise distance in a set with finite diameter is at most the diameter.
theorem distance_le_diameter_fn[M: MetricSpace](s: Set[M], x: M, y: M) {
    has_diameter(s) and s.contains(x) and s.contains(y) implies x.distance(y) <= diameter(s)
} by {
    if has_diameter(s) and s.contains(x) and s.contains(y) {
        diameter_is_diameter(s)
        is_diameter(s, diameter(s))
        distance_le_diameter(s, diameter(s), x, y)
    }
}

/// The diameter of a set is non-negative.
theorem diameter_non_negative_fn[M: MetricSpace](s: Set[M]) {
    not diameter(s).is_negative
} by {
    if has_diameter(s) {
        let x0: M satisfy { s.contains(x0) }
        diameter_is_diameter(s)
        diameter_non_negative(s, diameter(s), x0)
        not diameter(s).is_negative
    } else {
        diameter_default(s)
        not diameter(s).is_negative
    }
}

/// The diameter of the empty set is zero.
theorem diameter_empty[M: MetricSpace] {
    diameter(Set[M].empty_set) = 0
} by {
    if has_diameter(Set[M].empty_set) {
        has_diameter(Set[M].empty_set) = (is_bounded(Set[M].empty_set) and
            exists(x: M) { (Set[M].empty_set).contains(x) })
        exists(x: M) { (Set[M].empty_set).contains(x) }
        let x: M satisfy { (Set[M].empty_set).contains(x) }
        false
    }
    diameter_default(Set[M].empty_set)
}

/// A set of reals containing one of its upper bounds has that value as supremum.
theorem set_supremum_of_max(t: Set[Real], m: Real) {
    t.contains(m) and is_set_upper_bound(t, m) implies is_set_supremum(t, m)
} by {
    if t.contains(m) and is_set_upper_bound(t, m) {
        forall(c: Real) {
            if is_set_upper_bound(t, c) {
                is_set_upper_bound(t, c) = forall(z: Real) {
                    t.contains(z) implies z <= c
                }
                m <= c
            }
        }
        is_set_supremum(t, m)
    }
}

/// Zero is the diameter of a singleton set.
theorem is_diameter_singleton[M: MetricSpace](x: M) {
    is_diameter(Set[M].singleton(x), 0)
} by {
    let s: Set[M] = Set[M].singleton(x)
    forall(r: Real) {
        if distance_set(s).contains(r) {
            let a: M satisfy {
                exists(b: M) { s.contains(a) and s.contains(b) and r = a.distance(b) }
            }
            let b: M satisfy {
                s.contains(a) and s.contains(b) and r = a.distance(b)
            }
            s.contains(a) = (x = a)
            s.contains(b) = (x = b)
            x.distance(x) = 0
            r <= 0
        }
    }
    is_set_upper_bound(distance_set(s), 0)
    s.contains(x)
    distance_set_contains_distance(s, x, x)
    distance_set(s).contains(0)
    set_supremum_of_max(distance_set(s), 0)
}

/// The diameter of a singleton set is zero.
theorem diameter_singleton[M: MetricSpace](x: M) {
    diameter(Set[M].singleton(x)) = 0
} by {
    singleton_is_bounded(x)
    has_diameter(Set[M].singleton(x))
    diameter_is_diameter(Set[M].singleton(x))
    is_diameter_singleton(x)
    diameter_unique(Set[M].singleton(x), diameter(Set[M].singleton(x)), 0)
}

/// Pairwise-distance sets are monotone with respect to set inclusion.
theorem distance_set_subset_of_subset[M: MetricSpace](s: Set[M], t: Set[M]) {
    s.subset(t) implies distance_set(s).subset(distance_set(t))
} by {
    if s.subset(t) {
        forall(r: Real) {
            if distance_set(s).contains(r) {
                let x: M satisfy {
                    exists(y: M) { s.contains(x) and s.contains(y) and r = x.distance(y) }
                }
                let y: M satisfy {
                    s.contains(x) and s.contains(y) and r = x.distance(y)
                }
                subset_contains(s, t, x)
                subset_contains(s, t, y)
                distance_set_contains_distance(t, x, y)
                distance_set(t).contains(r)
            }
        }
    }
}

/// The diameter is monotone with respect to set inclusion.
theorem diameter_le_of_subset[M: MetricSpace](s: Set[M], t: Set[M]) {
    s.subset(t) and has_diameter(s) and has_diameter(t)
    implies diameter(s) <= diameter(t)
} by {
    if s.subset(t) and has_diameter(s) and has_diameter(t) {
        diameter_is_diameter(s)
        diameter_is_diameter(t)
        is_set_supremum(distance_set(s), diameter(s))
        is_set_supremum(distance_set(t), diameter(t))
        distance_set_subset_of_subset(s, t)
        supremum_subset_le(distance_set(s), distance_set(t), diameter(s), diameter(t))
        diameter(s) <= diameter(t)
    }
}


/// De Morgan: the complement of a union equals the intersection of complements.
theorem union_complement_eq_intersection_of_complements[M: MetricSpace](s: Set[M], t: Set[M]) {
    s.union(t).c = s.c.intersection(t.c)
} by {
    forall(x: M) {
        if s.union(t).c.contains(x) {
            not s.union(t).contains(x)
            s.c.contains(x)
            t.c.contains(x)
            s.c.intersection(t.c).contains(x)
        }
        if s.c.intersection(t.c).contains(x) {
            not s.contains(x)
            not t.contains(x)
            not s.union(t).contains(x)
            s.union(t).c.contains(x)
        }
        s.union(t).c.contains(x) = s.c.intersection(t.c).contains(x)
    }
    set_eq_of_contains_at_eq(s.union(t).c, s.c.intersection(t.c))
}

/// De Morgan: the complement of an intersection equals the union of complements.
theorem intersection_complement_eq_union_of_complements[M: MetricSpace](s: Set[M], t: Set[M]) {
    s.intersection(t).c = s.c.union(t.c)
} by {
    forall(x: M) {
        if s.intersection(t).c.contains(x) {
            not s.contains(x) or not t.contains(x)
            if not s.contains(x) {
                s.c.union(t.c).contains(x)
            } else {
                s.c.union(t.c).contains(x)
            }
        }
        if s.c.union(t.c).contains(x) {
            if s.c.contains(x) {
                not s.intersection(t).contains(x)
                s.intersection(t).c.contains(x)
            } else {
                not t.contains(x)
                not s.intersection(t).contains(x)
                s.intersection(t).c.contains(x)
            }
        }
        s.intersection(t).c.contains(x) = s.c.union(t.c).contains(x)
    }
    set_eq_of_contains_at_eq(s.intersection(t).c, s.c.union(t.c))
}

/// The intersection of two closed sets is closed.
theorem intersection_of_closed_metric_is_closed[M: MetricSpace](s: Set[M], t: Set[M]) {
    is_closed_metric(s) and is_closed_metric(t) implies is_closed_metric(s.intersection(t))
} by {
    if is_closed_metric(s) and is_closed_metric(t) {
        is_closed_metric(t) = is_open_metric(t.c)
        union_of_open_metric_is_open(s.c, t.c)
        is_open_metric(s.c.union(t.c))
        intersection_complement_eq_union_of_complements(s, t)
        is_closed_metric(s.intersection(t))
    }
}

/// The union of two closed sets is closed.
theorem union_of_closed_metric_is_closed[M: MetricSpace](s: Set[M], t: Set[M]) {
    is_closed_metric(s) and is_closed_metric(t) implies is_closed_metric(s.union(t))
} by {
    if is_closed_metric(s) and is_closed_metric(t) {
        is_closed_metric(t) = is_open_metric(t.c)
        intersection_of_open_metric_is_open(s.c, t.c)
        is_open_metric(s.c.intersection(t.c))
        union_complement_eq_intersection_of_complements(s, t)
        is_closed_metric(s.union(t))
    }
}
