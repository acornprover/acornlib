from nat import Nat
from real import Real
from real import only_abs_zero_eq_zero, converges_to, tail_bound
from real import triangle_ineq
from real import is_cauchy_seq, cauchy_imp_converges_to_limit, cauchy_imp_exists_limit
from analysis.metric.metric_space import MetricSpace, is_cauchy_metric, tendsto_metric, cauchy_metric_bound, metric_tail_bound, metric_tail_bound_at, cauchy_metric_bound_at, metric_tail_bound_from_distances, tendsto_metric_imp_cauchy
numerals Real

/// The induced distance on the real line is the absolute value of the difference.
define real_distance(x: Real, y: Real) -> Real {
    (x - y).abs
}

/// The absolute value of zero is zero.
theorem real_abs_zero {
    Real.0.abs = Real.0
} by {
}

/// The distance from a real to itself is zero.
theorem real_distance_self(x: Real) {
    real_distance(x, x) = 0
} by {
    (x - x).abs = Real.0.abs
}

/// Two reals at distance zero are equal.
theorem real_distance_zero_imp_eq(x: Real, y: Real) {
    real_distance(x, y) = 0 implies x = y
} by {
    if real_distance(x, y) = 0 {
        (x - y).abs = Real.0
        only_abs_zero_eq_zero(x - y)
        x - y = Real.0
        x + -y = Real.0
        x = y
    }
}

/// The induced real distance is symmetric.
theorem real_distance_symm(x: Real, y: Real) {
    real_distance(x, y) = real_distance(y, x)
} by {
    x - y = x + -y
    -x + y = y + -x
    -(x - y) = y - x
    (y - x).abs = (x - y).abs
}

/// The induced real distance satisfies the triangle inequality.
theorem real_distance_triangle(x: Real, y: Real, z: Real) {
    real_distance(x, z) <= real_distance(x, y) + real_distance(y, z)
} by {
    triangle_ineq(x - y, y - z)
    (x - y) + (y - z) = x + -y + (y + -z)
    (-y + y) + -z = Real.0 + -z
    x + -z = x - z
}

instance Real: MetricSpace {
    let distance: (Real, Real) -> Real = real_distance
}

/// A complete metric space is one in which every Cauchy sequence converges.
typeclass M: CompleteMetricSpace extends MetricSpace {
    /// Rule: every Cauchy sequence has a limit.
    cauchy_metric_imp_tendsto(q: Nat -> M) {
        is_cauchy_metric(q) implies exists(a: M) {
            tendsto_metric(q, a)
        }
    }
}

/// A metric tail bound for real-valued sequences is a classical real tail bound.
theorem real_metric_tail_bound_imp_tail_bound(q: Nat -> Real, a: Real, n: Nat, eps: Real) {
    metric_tail_bound(q, a, n, eps) implies tail_bound(q, a, n, eps)
} by {
    if metric_tail_bound(q, a, n, eps) {
        forall(i: Nat) {
            if n <= i {
                metric_tail_bound_at(q, a, n, eps, i)
                q(i).distance(a) < eps
                q(i).distance(a) = real_distance(q(i), a)
                (q(i) - a).abs < eps
                q(i).is_close(a, eps)
            }
        }
        tail_bound(q, a, n, eps)
    }
}

/// A real sequence convergent in the metric sense converges in the classical sense.
theorem real_tendsto_metric_imp_converges_to(q: Nat -> Real, a: Real) {
    tendsto_metric(q, a) implies converges_to(q, a)
} by {
    if tendsto_metric(q, a) {
        forall(eps: Real) {
            if eps.is_positive {
                let n: Nat satisfy {
                    metric_tail_bound(q, a, n, eps)
                }
                real_metric_tail_bound_imp_tail_bound(q, a, n, eps)
                exists(m: Nat) { tail_bound(q, a, m, eps) }
            }
        }
    }
}

/// A real sequence convergent in the Weierstrass sense converges in the metric sense.
theorem real_converges_to_imp_tendsto_metric(q: Nat -> Real, a: Real) {
    converges_to(q, a) implies tendsto_metric(q, a)
} by {
    if converges_to(q, a) {
        forall(eps: Real) {
            if eps.is_positive {
                let n: Nat satisfy {
                    tail_bound(q, a, n, eps)
                }
                tail_bound(q, a, n, eps) = forall(i: Nat) {
                    n <= i implies q(i).is_close(a, eps)
                }
                forall(i: Nat) {
                    if n <= i {
                        (q(i) - a).abs < eps
                        real_distance(q(i), a) = (q(i) - a).abs
                        q(i).distance(a) < eps
                    }
                }
                metric_tail_bound_from_distances(q, a, n, eps)
                exists(m: Nat) { metric_tail_bound(q, a, m, eps) }
            }
        }
    }
}

/// A real metric Cauchy bound is a classical pointwise Cauchy bound.
theorem real_cauchy_metric_bound_imp_close_bound(q: Nat -> Real, n: Nat, eps: Real) {
    cauchy_metric_bound(q, n, eps) implies forall(i: Nat, j: Nat) {
        n <= i and n <= j implies q(i).is_close(q(j), eps)
    }
} by {
    if cauchy_metric_bound(q, n, eps) {
        forall(i: Nat, j: Nat) {
            if n <= i and n <= j {
                cauchy_metric_bound_at(q, n, eps, i, j)
                q(i).distance(q(j)) < eps
                q(i).distance(q(j)) = real_distance(q(i), q(j))
                (q(i) - q(j)).abs < eps
                q(i).is_close(q(j), eps)
            }
        }
    }
}

/// A classically Cauchy real sequence is Cauchy in the metric sense.
theorem real_is_cauchy_seq_imp_is_cauchy_metric(q: Nat -> Real) {
    is_cauchy_seq(q) implies is_cauchy_metric(q)
} by {
    if is_cauchy_seq(q) {
        cauchy_imp_exists_limit(q)
        let a: Real satisfy {
            converges_to(q, a)
        }
        real_converges_to_imp_tendsto_metric(q, a)
        tendsto_metric_imp_cauchy(q, a)
        is_cauchy_metric(q)
    }
}

/// A real Cauchy metric sequence is a Cauchy sequence in the `is_cauchy_seq` sense.
theorem real_is_cauchy_metric_imp_is_cauchy_seq(q: Nat -> Real) {
    is_cauchy_metric(q) implies is_cauchy_seq(q)
} by {
    if is_cauchy_metric(q) {
        forall(eps: Real) {
            if eps.is_positive {
                let n: Nat satisfy {
                    cauchy_metric_bound(q, n, eps)
                }
                forall(i: Nat, j: Nat) {
                    if n <= i and n <= j {
                        cauchy_metric_bound_at(q, n, eps, i, j)
                        q(i).distance(q(j)) < eps
                        q(i).distance(q(j)) = real_distance(q(i), q(j))
                        (q(i) - q(j)).abs < eps
                        q(i).is_close(q(j), eps)
                    }
                }
                exists(m: Nat) {
                    forall(i: Nat, j: Nat) {
                        m <= i and m <= j implies q(i).is_close(q(j), eps)
                    }
                }
            }
        }
    }
}

/// Classical and metric Cauchy predicates agree for real-valued sequences.
theorem real_is_cauchy_metric_iff_is_cauchy_seq(q: Nat -> Real) {
    is_cauchy_metric(q) = is_cauchy_seq(q)
} by {
    if is_cauchy_metric(q) {
        real_is_cauchy_metric_imp_is_cauchy_seq(q)
        is_cauchy_seq(q)
    }
    if is_cauchy_seq(q) {
        real_is_cauchy_seq_imp_is_cauchy_metric(q)
        is_cauchy_metric(q)
    }
}

/// Classical and metric convergence agree for real-valued sequences.
theorem real_tendsto_metric_iff_converges_to(q: Nat -> Real, a: Real) {
    tendsto_metric(q, a) = converges_to(q, a)
} by {
    if tendsto_metric(q, a) {
        real_tendsto_metric_imp_converges_to(q, a)
        converges_to(q, a)
    }
    if converges_to(q, a) {
        real_converges_to_imp_tendsto_metric(q, a)
        tendsto_metric(q, a)
    }
}

/// Every Cauchy real sequence has a metric limit.
theorem real_cauchy_metric_imp_tendsto(q: Nat -> Real) {
    is_cauchy_metric(q) implies exists(a: Real) {
        tendsto_metric(q, a)
    }
} by {
    if is_cauchy_metric(q) {
        real_is_cauchy_metric_imp_is_cauchy_seq(q)
        cauchy_imp_converges_to_limit(q)
        let a: Real satisfy {
            converges_to(q, a)
        }
        real_converges_to_imp_tendsto_metric(q, a)
        tendsto_metric(q, a)
    }
}

instance Real: CompleteMetricSpace
