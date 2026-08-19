/// Foundations of Fourier series: the orthogonality of the trigonometric
/// system on [0, 2*pi], and the algebraic identities behind it.
///
/// The results are:
///   - the vanishing of the integrals of sine and cosine over one full
///     period (the simplest orthogonality instances),
///   - the double-angle and half-angle identities,
///   - the product-to-sum identities, which turn a product of two harmonics
///     into a sum of two single harmonics (the algebraic heart of
///     orthogonality),
///   - boundedness, two-Lipschitzness, and continuity of Real.sin * Real.cos,
///   - the integral of Real.sin * Real.cos over [0, 2*pi] is zero, computed from the
///     fundamental theorem of calculus with antiderivative Real.sin^2 / 2,
///   - the Dirichlet-kernel summation identity
///     2 (x/2).sin * (1/2 + sum_{k=1}^{n} (k x).cos) = ((n + 1/2) x).sin.
///
/// The concrete integral values of x.sin^2, x.cos^2, x.sin (2x).sin, and
/// the first sine Fourier coefficient b_1 = -2 of the sawtooth x on
/// [0, 2*pi] are proved in the sibling module analysis/fourier_basics.ac;
/// the two modules together cover the orthogonality and normalization of
/// the first few harmonics.

from nat import Nat, from_nat, from_nat_zero, from_nat_add, pow_one, alt_induction
from real import Real, pi, pi_pos, pi_over_two, two, one_half_plus_one_half,
    sin_zero, cos_zero, sin_pi_zero, cos_pi_neg_one, sin_two_pi_zero,
    cos_two_pi_one, sin_add, cos_add, sin_sq_add_cos_sq, sin_neg, cos_neg,
    sin_continuous, cos_continuous, sin_lipschitz, cos_lipschitz,
    sin_abs_le_one, cos_abs_le_one, integral, integral_sin, integral_cos,
    integral_sin_zero_pi, is_integrable, continuous_pointwise_mul, continuous,
    interval_contains, mul_abs, abs_gte_zero, lte_abs, abs_neg, neg_neg,
    neg_distrib, lte_trans, lt_trans, lt_lte_trans, lte_lt_trans, lte_add_right,
    triangle_ineq, pos_gt_zero, gt_zero_imp_pos, lt_add_pos, is_derivative_fn,
    derivative_fn_square, derivative_fn_const_mul, sin_is_derivative_fn,
    ftc2_general, fn_integrable_gen, two_positive, continuous_const_mul_left,
    const_mul_left, neg_lte_flip, real_mul_comm, mul_assoc, mul_distrib_left,
    add_assoc, add_comm, add_neg_eq_zero, add_zero_left, add_zero_right,
    sub_cancels, sub_moves_sides, one_half_positive, mul_sub_distrib_right,
    mul_sub_distrib_left, mul_neg_one_right
from data.basic.functions import compose, identity_fn, function_extensionality
from data.basic.function_algebra import pointwise_mul, pointwise_add, pointwise_neg,
    pointwise_mul_apply, pointwise_add_apply, pointwise_neg_apply
from list import partial, partial_add, partial_scalar_mul, partial_pointwise_eq
from algebra.add_ordered_group import add_le_add
from algebra.semigroup import mul_fn
from ordered_field import mul_le_mul_of_nonneg_right, mul_le_mul_of_nonneg_left
from order import lt_imp_lte, lte_antisymm

numerals Real
numerals Nat

// ---------------------------------------------------------------------------
// The period 2*pi, and special values at the period
// ---------------------------------------------------------------------------

/// Two pi equals pi plus pi.
theorem two_pi_eq_pi_add_pi {
    two * pi = pi + pi
} by {
    two = Real.1 + Real.1
    two * pi = (Real.1 + Real.1) * pi
    (Real.1 + Real.1) * pi = pi + pi
    two * pi = pi + pi
}

/// The period 2*pi is positive.
theorem two_pi_pos {
    Real.0 < two * pi
} by {
    two_pi_eq_pi_add_pi
    two * pi = pi + pi
    pi_pos
    pi > Real.0
    gt_zero_imp_pos(pi)
    pi.is_positive
    lt_add_pos(pi, pi)
    pi.is_positive implies pi < pi + pi
    pi < pi + pi
    lt_trans(Real.0, pi, pi + pi)
    Real.0 < pi + pi
    two * pi = pi + pi
    Real.0 < two * pi
}

/// The sine of the period 2*pi is zero.
theorem sin_two_pi_value {
    (two * pi).sin = Real.0
} by {
    two_pi_eq_pi_add_pi
    two * pi = pi + pi
    (two * pi).sin = (pi + pi).sin
    sin_two_pi_zero
    (pi + pi).sin = Real.0
    (two * pi).sin = Real.0
}

/// The cosine of the period 2*pi is one.
theorem cos_two_pi_value {
    (two * pi).cos = Real.1
} by {
    two_pi_eq_pi_add_pi
    two * pi = pi + pi
    (two * pi).cos = (pi + pi).cos
    cos_two_pi_one
    (pi + pi).cos = Real.1
    (two * pi).cos = Real.1
}

// ---------------------------------------------------------------------------
// Integrals of sine and cosine over periods: the first orthogonality
// instances (the constant function 1 is orthogonal to Real.sin and Real.cos on a full
// period, and to Real.cos on [0, pi]).
// ---------------------------------------------------------------------------

/// The integral of sine over one full period [0, 2*pi] is zero.
theorem integral_sin_zero_two_pi {
    integral(Real.sin, Real.0, two * pi) = Real.0
} by {
    two_pi_pos
    Real.0 < two * pi
    lt_imp_lte(Real.0, two * pi)
    Real.0 <= two * pi
    integral_sin(Real.0, two * pi)
    Real.0 <= two * pi implies integral(Real.sin, Real.0, two * pi) = (Real.0).cos - (two * pi).cos
    integral(Real.sin, Real.0, two * pi) = (Real.0).cos - (two * pi).cos
    cos_zero
    (Real.0).cos = Real.1
    cos_two_pi_value
    (two * pi).cos = Real.1
    (Real.0).cos - (two * pi).cos = Real.1 - Real.1
    Real.1 - Real.1 = Real.0
    integral(Real.sin, Real.0, two * pi) = Real.0
}

/// The integral of cosine over one full period [0, 2*pi] is zero.
theorem integral_cos_zero_two_pi {
    integral(Real.cos, Real.0, two * pi) = Real.0
} by {
    two_pi_pos
    Real.0 < two * pi
    lt_imp_lte(Real.0, two * pi)
    Real.0 <= two * pi
    integral_cos(Real.0, two * pi)
    Real.0 <= two * pi implies integral(Real.cos, Real.0, two * pi) = (two * pi).sin - (Real.0).sin
    integral(Real.cos, Real.0, two * pi) = (two * pi).sin - (Real.0).sin
    sin_two_pi_value
    (two * pi).sin = Real.0
    sin_zero
    (Real.0).sin = Real.0
    (two * pi).sin - (Real.0).sin = Real.0 - Real.0
    Real.0 - Real.0 = Real.0
    integral(Real.cos, Real.0, two * pi) = Real.0
}

/// The integral of cosine over [0, pi] is zero.
theorem integral_cos_zero_pi {
    integral(Real.cos, Real.0, pi) = Real.0
} by {
    pi_pos
    pi > Real.0
    lt_imp_lte(Real.0, pi)
    Real.0 <= pi
    integral_cos(Real.0, pi)
    Real.0 <= pi implies integral(Real.cos, Real.0, pi) = pi.sin - (Real.0).sin
    integral(Real.cos, Real.0, pi) = pi.sin - (Real.0).sin
    sin_pi_zero
    pi.sin = Real.0
    sin_zero
    (Real.0).sin = Real.0
    pi.sin - (Real.0).sin = Real.0 - Real.0
    Real.0 - Real.0 = Real.0
    integral(Real.cos, Real.0, pi) = Real.0
}

/// The integral of sine over [0, pi] is two (restated for this module).
theorem integral_sin_zero_pi_restated {
    integral(Real.sin, Real.0, pi) = two
} by {
    integral_sin_zero_pi
}

// ---------------------------------------------------------------------------
// Double-angle and half-angle identities
// ---------------------------------------------------------------------------

/// The double-angle identity for sine: (2x).sin = 2 x.sin x.cos.
theorem sin_double(x: Real) {
    (two * x).sin = two * x.sin * x.cos
} by {
    two * x = x + x
    (two * x).sin = (x + x).sin
    sin_add(x, x)
    (x + x).sin = x.sin * x.cos + x.cos * x.sin
    real_mul_comm(x.cos, x.sin)
    x.cos * x.sin = x.sin * x.cos
    x.sin * x.cos + x.cos * x.sin = x.sin * x.cos + x.sin * x.cos
    x.sin * x.cos + x.sin * x.cos = two * (x.sin * x.cos)
    mul_assoc(two, x.sin, x.cos)
    two * (x.sin * x.cos) = two * x.sin * x.cos
    x.sin * x.cos + x.sin * x.cos = two * x.sin * x.cos
    (two * x).sin = two * x.sin * x.cos
}

/// The double-angle identity for cosine:
/// (2x).cos = x.cos * x.cos - x.sin * x.sin.
theorem cos_double(x: Real) {
    (two * x).cos = x.cos * x.cos - x.sin * x.sin
} by {
    two * x = x + x
    (two * x).cos = (x + x).cos
    cos_add(x, x)
    (x + x).cos = x.cos * x.cos - x.sin * x.sin
    (two * x).cos = x.cos * x.cos - x.sin * x.sin
}

/// Ring helper: (a - b) + b = a, via unfolding the subtraction.
theorem sub_add_cancel(a: Real, b: Real) {
    (a - b) + b = a
} by {
    add_neg_eq_zero(b)
    b + -b = Real.0
    add_comm(b, -b)
    -b + b = Real.0
    (a - b) + b = a + -b + b
    a + -b + b = a + (-b + b)
    a + (-b + b) = a + Real.0
    add_zero_right(a)
    a + Real.0 = a
    a + (-b + b) = a
    (a - b) + b = a
}

/// Ring helper: telescoping of subtractions, (a - b) + (b - c) = a - c.
theorem add_sub_sub_cancel(a: Real, b: Real, c: Real) {
    (a - b) + (b - c) = a - c
} by {
    add_assoc(a - b, b, -c)
    ((a - b) + b) + -c = (a - b) + (b + -c)
    (a - b) + (b - c) = ((a - b) + b) + -c
    sub_add_cancel(a, b)
    (a - b) + b = a
    ((a - b) + b) + -c = a + -c
    a + -c = a - c
    (a - b) + (b - c) = a - c
}

/// Ring helper: (a + b) - (b + c) = a - c.
theorem add_sub_add_cancel(a: Real, b: Real, c: Real) {
    (a + b) - (b + c) = a - c
} by {
    add_assoc(a, b, -b)
    (a + b) + -b = a + (b + -b)
    add_neg_eq_zero(b)
    b + -b = Real.0
    a + (b + -b) = a + Real.0
    add_zero_right(a)
    a + Real.0 = a
    (a + b) + -b = a
    add_assoc(a + b, -b, -c)
    ((a + b) + -b) + -c = (a + b) + (-b + -c)
    neg_distrib(b, c)
    -(b + c) = -b + -c
    (a + b) - (b + c) = (a + b) + (-b + -c)
    ((a + b) + -b) + -c = a + -c
    a + -c = a - c
    (a + b) - (b + c) = a - c
}

/// Ring identity: a + b - (b - a) = a + a, used by the half-angle
/// identities below (the ring solver times out on it directly, so the
/// subtraction is unfolded explicitly).
theorem ring_add_sub_comm(a: Real, b: Real) {
    a + b - (b - a) = a + a
} by {
    a - b = a + (-b)
    a + b - (b - a) = a + b + (-(b - a))
    neg_distrib(b, -a)
    -(b + (-a)) = -b + -(-a)
    neg_neg(a)
    -(-a) = a
    b - a = b + (-a)
    -(b - a) = -b + a
    a + b + (-(b - a)) = a + b + (-b + a)
    add_assoc(a, b, -b + a)
    a + b + (-b + a) = a + (b + (-b + a))
    add_assoc(b, -b, a)
    b + (-b + a) = (b + (-b)) + a
    a + (b + (-b + a)) = a + ((b + (-b)) + a)
    add_neg_eq_zero(b)
    b + (-b) = Real.0
    a + ((b + (-b)) + a) = a + (Real.0 + a)
    add_zero_left(a)
    Real.0 + a = a
    a + (Real.0 + a) = a + a
    a + b + (-b + a) = a + a
    a + b + (-(b - a)) = a + a
    a + b - (b - a) = a + a
}


/// Ring identity: (a + b) - (a - b) = b + b, used to combine the two
/// halves of the sine addition law into 2 a.cos b.sin.
theorem ring_sub_sub_comm(a: Real, b: Real) {
    (a + b) - (a - b) = b + b
} by {
    a - b = a + (-b)
    (a + b) - (a - b) = (a + b) + (-(a - b))
    neg_distrib(a, -b)
    -(a + (-b)) = -a + -(-b)
    neg_neg(b)
    -(-b) = b
    -(a - b) = -a + b
    (a + b) + (-(a - b)) = (a + b) + (-a + b)
    add_comm(a, b)
    a + b = b + a
    (a + b) + (-a + b) = (b + a) + (-a + b)
    add_assoc(b, a, -a + b)
    (b + a) + (-a + b) = b + (a + (-a + b))
    add_assoc(a, -a, b)
    a + (-a + b) = (a + -a) + b
    b + (a + (-a + b)) = b + ((a + -a) + b)
    add_neg_eq_zero(a)
    a + -a = Real.0
    b + ((a + -a) + b) = b + (Real.0 + b)
    add_zero_left(b)
    Real.0 + b = b
    b + (Real.0 + b) = b + b
    (a + b) - (a - b) = b + b
}

/// Ring identity: a + b + (a - b) = a + a, the companion of
/// ring_add_sub_comm (same explicit unfolding of the subtraction).
theorem ring_add_sub_comm_b(a: Real, b: Real) {
    a + b + (a - b) = a + a
} by {
    a - b = a + (-b)
    a + b + (a - b) = a + b + (a + (-b))
    add_assoc(a + b, a, -b)
    a + b + (a + (-b)) = (a + b + a) + (-b)
    add_comm(a + b, a)
    (a + b + a) + (-b) = (a + a + b) + (-b)
    add_comm(a + a, b)
    (a + a + b) + (-b) = (b + (a + a)) + (-b)
    add_assoc(b, a + a, -b)
    (b + (a + a)) + (-b) = b + ((a + a) + (-b))
    add_comm(a + a, -b)
    b + ((a + a) + (-b)) = b + ((-b) + (a + a))
    add_assoc(b, -b, a + a)
    b + ((-b) + (a + a)) = (b + (-b)) + (a + a)
    add_neg_eq_zero(b)
    b + (-b) = Real.0
    (b + (-b)) + (a + a) = Real.0 + (a + a)
    add_zero_left(a + a)
    Real.0 + (a + a) = a + a
    a + b + (a - b) = a + a
}

/// 2 x.sin^2 = 1 - (2x).cos, the half-angle identity for sine.
theorem sin_sq_times_two(x: Real) {
    two * (x.sin * x.sin) = Real.1 - (two * x).cos
} by {
    sin_sq_add_cos_sq(x)
    x.sin.pow(Nat.2) + x.cos.pow(Nat.2) = Real.1
    x.sin.pow(Nat.2) = x.sin * x.sin
    x.cos.pow(Nat.2) = x.cos * x.cos
    x.sin * x.sin + x.cos * x.cos = Real.1
    cos_double(x)
    (two * x).cos = x.cos * x.cos - x.sin * x.sin
    Real.1 - (two * x).cos = Real.1 - (x.cos * x.cos - x.sin * x.sin)
    Real.1 = x.sin * x.sin + x.cos * x.cos
    Real.1 - (x.cos * x.cos - x.sin * x.sin) = (x.sin * x.sin + x.cos * x.cos) - (x.cos * x.cos - x.sin * x.sin)
    ring_add_sub_comm(x.sin * x.sin, x.cos * x.cos)
    (x.sin * x.sin + x.cos * x.cos) - (x.cos * x.cos - x.sin * x.sin) = x.sin * x.sin + x.sin * x.sin
    x.sin * x.sin + x.sin * x.sin = two * (x.sin * x.sin)
    Real.1 - (x.cos * x.cos - x.sin * x.sin) = two * (x.sin * x.sin)
    two * (x.sin * x.sin) = Real.1 - (two * x).cos
}

/// 2 x.cos^2 = 1 + (2x).cos, the half-angle identity for cosine.
theorem cos_sq_times_two(x: Real) {
    two * (x.cos * x.cos) = Real.1 + (two * x).cos
} by {
    sin_sq_add_cos_sq(x)
    x.sin.pow(Nat.2) + x.cos.pow(Nat.2) = Real.1
    x.sin.pow(Nat.2) = x.sin * x.sin
    x.cos.pow(Nat.2) = x.cos * x.cos
    x.sin * x.sin + x.cos * x.cos = Real.1
    cos_double(x)
    (two * x).cos = x.cos * x.cos - x.sin * x.sin
    Real.1 + (two * x).cos = Real.1 + (x.cos * x.cos - x.sin * x.sin)
    Real.1 = x.sin * x.sin + x.cos * x.cos
    Real.1 + (x.cos * x.cos - x.sin * x.sin) = (x.sin * x.sin + x.cos * x.cos) + (x.cos * x.cos - x.sin * x.sin)
    add_comm(x.sin * x.sin, x.cos * x.cos)
    x.sin * x.sin + x.cos * x.cos = x.cos * x.cos + x.sin * x.sin
    (x.sin * x.sin + x.cos * x.cos) + (x.cos * x.cos - x.sin * x.sin) = (x.cos * x.cos + x.sin * x.sin) + (x.cos * x.cos - x.sin * x.sin)
    ring_add_sub_comm_b(x.cos * x.cos, x.sin * x.sin)
    (x.cos * x.cos + x.sin * x.sin) + (x.cos * x.cos - x.sin * x.sin) = x.cos * x.cos + x.cos * x.cos
    (x.sin * x.sin + x.cos * x.cos) + (x.cos * x.cos - x.sin * x.sin) = x.cos * x.cos + x.cos * x.cos
    x.cos * x.cos + x.cos * x.cos = two * (x.cos * x.cos)
    Real.1 + (x.cos * x.cos - x.sin * x.sin) = two * (x.cos * x.cos)
    two * (x.cos * x.cos) = Real.1 + (two * x).cos
}

// ---------------------------------------------------------------------------
// Product-to-sum identities: the algebraic heart of orthogonality.  A
// product of two harmonics is a sum of two single harmonics.
// ---------------------------------------------------------------------------

/// 2 x.sin y.cos = (x + y).sin + (x - y).sin.
theorem product_to_sum_sin_cos(x: Real, y: Real) {
    two * x.sin * y.cos = (x + y).sin + (x - y).sin
} by {
    sin_add(x, y)
    (x + y).sin = x.sin * y.cos + x.cos * y.sin
    sin_add(x, -y)
    (x + (-y)).sin = x.sin * (-y).cos + x.cos * (-y).sin
    x + (-y) = x - y
    (x - y).sin = x.sin * (-y).cos + x.cos * (-y).sin
    cos_neg(y)
    (-y).cos = y.cos
    sin_neg(y)
    (-y).sin = -y.sin
    x.sin * (-y).cos + x.cos * (-y).sin = x.sin * y.cos - x.cos * y.sin
    (x - y).sin = x.sin * y.cos - x.cos * y.sin
    (x + y).sin + (x - y).sin = x.sin * y.cos + x.cos * y.sin + (x.sin * y.cos - x.cos * y.sin)
    ring_add_sub_comm_b(x.sin * y.cos, x.cos * y.sin)
    x.sin * y.cos + x.cos * y.sin + (x.sin * y.cos - x.cos * y.sin) = x.sin * y.cos + x.sin * y.cos
    x.sin * y.cos + x.sin * y.cos = two * x.sin * y.cos
    (x + y).sin + (x - y).sin = two * x.sin * y.cos
    two * x.sin * y.cos = (x + y).sin + (x - y).sin
}

/// 2 x.sin y.sin = (x - y).cos - (x + y).cos.
theorem product_to_sum_sin_sin(x: Real, y: Real) {
    two * x.sin * y.sin = (x - y).cos - (x + y).cos
} by {
    cos_add(x, y)
    (x + y).cos = x.cos * y.cos - x.sin * y.sin
    cos_add(x, -y)
    (x + (-y)).cos = x.cos * (-y).cos - x.sin * (-y).sin
    x + (-y) = x - y
    (x - y).cos = x.cos * (-y).cos - x.sin * (-y).sin
    cos_neg(y)
    (-y).cos = y.cos
    sin_neg(y)
    (-y).sin = -y.sin
    x.cos * (-y).cos - x.sin * (-y).sin = x.cos * y.cos + x.sin * y.sin
    (x - y).cos = x.cos * y.cos + x.sin * y.sin
    (x - y).cos - (x + y).cos = (x.cos * y.cos + x.sin * y.sin) - (x.cos * y.cos - x.sin * y.sin)
    ring_add_sub_comm(x.sin * y.sin, x.cos * y.cos)
    (x.sin * y.sin + x.cos * y.cos) - (x.cos * y.cos - x.sin * y.sin) = x.sin * y.sin + x.sin * y.sin
    add_comm(x.sin * y.sin, x.cos * y.cos)
    x.sin * y.sin + x.cos * y.cos = x.cos * y.cos + x.sin * y.sin
    (x.cos * y.cos + x.sin * y.sin) - (x.cos * y.cos - x.sin * y.sin) = x.sin * y.sin + x.sin * y.sin
    x.sin * y.sin + x.sin * y.sin = two * x.sin * y.sin
    (x - y).cos - (x + y).cos = two * x.sin * y.sin
    two * x.sin * y.sin = (x - y).cos - (x + y).cos
}

/// 2 x.cos y.cos = (x + y).cos + (x - y).cos.
theorem product_to_sum_cos_cos(x: Real, y: Real) {
    two * x.cos * y.cos = (x + y).cos + (x - y).cos
} by {
    cos_add(x, y)
    (x + y).cos = x.cos * y.cos - x.sin * y.sin
    cos_add(x, -y)
    (x + (-y)).cos = x.cos * (-y).cos - x.sin * (-y).sin
    x + (-y) = x - y
    (x - y).cos = x.cos * (-y).cos - x.sin * (-y).sin
    cos_neg(y)
    (-y).cos = y.cos
    sin_neg(y)
    (-y).sin = -y.sin
    x.cos * (-y).cos - x.sin * (-y).sin = x.cos * y.cos + x.sin * y.sin
    (x - y).cos = x.cos * y.cos + x.sin * y.sin
    (x + y).cos + (x - y).cos = (x.cos * y.cos - x.sin * y.sin) + (x.cos * y.cos + x.sin * y.sin)
    add_comm(x.cos * y.cos - x.sin * y.sin, x.cos * y.cos + x.sin * y.sin)
    (x.cos * y.cos - x.sin * y.sin) + (x.cos * y.cos + x.sin * y.sin) = (x.cos * y.cos + x.sin * y.sin) + (x.cos * y.cos - x.sin * y.sin)
    ring_add_sub_comm_b(x.cos * y.cos, x.sin * y.sin)
    (x.cos * y.cos + x.sin * y.sin) + (x.cos * y.cos - x.sin * y.sin) = x.cos * y.cos + x.cos * y.cos
    x.cos * y.cos + x.cos * y.cos = two * x.cos * y.cos
    (x + y).cos + (x - y).cos = two * x.cos * y.cos
    two * x.cos * y.cos = (x + y).cos + (x - y).cos
}

// ---------------------------------------------------------------------------
// Boundedness, Lipschitz, and continuity of Real.sin * Real.cos
// ---------------------------------------------------------------------------

/// The product Real.sin * Real.cos.
define sincos(x: Real) -> Real {
    x.sin * x.cos
}

/// The product Real.sin * Real.cos is bounded in absolute value by one.
theorem sincos_abs_le_one(x: Real) {
    sincos(x).abs <= Real.1
} by {
    sincos(x) = x.sin * x.cos
    sin_abs_le_one(x)
    x.sin.abs <= Real.1
    cos_abs_le_one(x)
    x.cos.abs <= Real.1
    mul_abs(x.sin, x.cos)
    (x.sin * x.cos).abs = x.sin.abs * x.cos.abs
    abs_gte_zero(x.cos)
    Real.0 <= x.cos.abs
    mul_le_mul_of_nonneg_right(x.sin.abs, Real.1, x.cos.abs)
    x.sin.abs * x.cos.abs <= Real.1 * x.cos.abs
    Real.1 * x.cos.abs = x.cos.abs
    x.sin.abs * x.cos.abs <= x.cos.abs
    lte_trans(x.sin.abs * x.cos.abs, x.cos.abs, Real.1)
    sincos(x).abs <= Real.1
}

/// The product Real.sin * Real.cos is at most one.
theorem sincos_le_one(x: Real) {
    sincos(x) <= Real.1
} by {
    sincos_abs_le_one(x)
    sincos(x).abs <= Real.1
    lte_abs(sincos(x))
    sincos(x) <= sincos(x).abs
    lte_trans(sincos(x), sincos(x).abs, Real.1)
    sincos(x) <= Real.1
}

/// The product Real.sin * Real.cos is at least negative one.
theorem sincos_ge_neg_one(x: Real) {
    -Real.1 <= sincos(x)
} by {
    sincos_abs_le_one(x)
    sincos(x).abs <= Real.1
    lte_abs(-sincos(x))
    -sincos(x) <= (-sincos(x)).abs
    abs_neg(sincos(x))
    (-sincos(x)).abs = sincos(x).abs
    -sincos(x) <= sincos(x).abs
    lte_trans(-sincos(x), sincos(x).abs, Real.1)
    -sincos(x) <= Real.1
    neg_lte_flip(-sincos(x), Real.1)
    -Real.1 <= -(-sincos(x))
    neg_neg(sincos(x))
    -(-sincos(x)) = sincos(x)
    -Real.1 <= sincos(x)
}

/// Real.sin * Real.cos is two-Lipschitz: |(u).sincos(u) - (v).sincos(v)| <= 2 |u - v|.
theorem sincos_lipschitz_m2(u: Real, v: Real) {
    (sincos(u) - sincos(v)).abs <= two * (u - v).abs
} by {
    sincos(u) = u.sin * u.cos
    sincos(v) = v.sin * v.cos
    sincos(u) - sincos(v) = u.sin * u.cos - v.sin * v.cos
    (sincos(u) - sincos(v)).abs <= (u.sin * u.cos - v.sin * v.cos).abs
    mul_sub_distrib_right(u.sin, u.cos, v.cos)
    u.sin * (u.cos - v.cos) = u.sin * u.cos - u.sin * v.cos
    mul_sub_distrib_left(u.sin, v.sin, v.cos)
    (u.sin - v.sin) * v.cos = u.sin * v.cos - v.sin * v.cos
    real_mul_comm(u.sin - v.sin, v.cos)
    (u.sin - v.sin) * v.cos = v.cos * (u.sin - v.sin)
    real_mul_comm(v.sin, v.cos)
    v.sin * v.cos = v.cos * v.sin
    u.sin * (u.cos - v.cos) + v.cos * (u.sin - v.sin) =
        u.sin * u.cos - u.sin * v.cos + (u.sin * v.cos - v.sin * v.cos)
    u.sin * u.cos - u.sin * v.cos + (u.sin * v.cos - v.sin * v.cos) =
        u.sin * u.cos - u.sin * v.cos + (u.sin * v.cos - v.cos * v.sin)
    add_sub_sub_cancel(u.sin * u.cos, u.sin * v.cos, v.cos * v.sin)
    (u.sin * u.cos - u.sin * v.cos) + (u.sin * v.cos - v.cos * v.sin) = u.sin * u.cos - v.cos * v.sin
    u.sin * u.cos - u.sin * v.cos + (u.sin * v.cos - v.cos * v.sin) = u.sin * u.cos - v.cos * v.sin
    u.sin * (u.cos - v.cos) + v.cos * (u.sin - v.sin) = u.sin * u.cos - v.cos * v.sin
    u.sin * u.cos - v.sin * v.cos = u.sin * u.cos - v.cos * v.sin
    u.sin * (u.cos - v.cos) + v.cos * (u.sin - v.sin) = u.sin * u.cos - v.sin * v.cos
    u.sin * u.cos - v.sin * v.cos = u.sin * (u.cos - v.cos) + v.cos * (u.sin - v.sin)
    (u.sin * u.cos - v.sin * v.cos).abs <= (u.sin * (u.cos - v.cos) + v.cos * (u.sin - v.sin)).abs
    triangle_ineq(u.sin * (u.cos - v.cos), v.cos * (u.sin - v.sin))
    (u.sin * (u.cos - v.cos) + v.cos * (u.sin - v.sin)).abs <= (u.sin * (u.cos - v.cos)).abs + (v.cos * (u.sin - v.sin)).abs
    mul_abs(u.sin, u.cos - v.cos)
    (u.sin * (u.cos - v.cos)).abs = u.sin.abs * (u.cos - v.cos).abs
    mul_abs(v.cos, u.sin - v.sin)
    (v.cos * (u.sin - v.sin)).abs = v.cos.abs * (u.sin - v.sin).abs
    (u.sin * (u.cos - v.cos) + v.cos * (u.sin - v.sin)).abs <= u.sin.abs * (u.cos - v.cos).abs + v.cos.abs * (u.sin - v.sin).abs
    sin_abs_le_one(u)
    u.sin.abs <= Real.1
    cos_lipschitz(u, v)
    (u.cos - v.cos).abs <= (u - v).abs
    abs_gte_zero(u.cos - v.cos)
    Real.0 <= (u.cos - v.cos).abs
    mul_le_mul_of_nonneg_right(u.sin.abs, Real.1, (u.cos - v.cos).abs)
    u.sin.abs * (u.cos - v.cos).abs <= Real.1 * (u.cos - v.cos).abs
    Real.1 * (u.cos - v.cos).abs = (u.cos - v.cos).abs
    u.sin.abs * (u.cos - v.cos).abs <= (u.cos - v.cos).abs
    lte_trans(u.sin.abs * (u.cos - v.cos).abs, (u.cos - v.cos).abs, (u - v).abs)
    u.sin.abs * (u.cos - v.cos).abs <= (u - v).abs
    cos_abs_le_one(v)
    v.cos.abs <= Real.1
    sin_lipschitz(u, v)
    (u.sin - v.sin).abs <= (u - v).abs
    abs_gte_zero(u.sin - v.sin)
    Real.0 <= (u.sin - v.sin).abs
    mul_le_mul_of_nonneg_right(v.cos.abs, Real.1, (u.sin - v.sin).abs)
    v.cos.abs * (u.sin - v.sin).abs <= Real.1 * (u.sin - v.sin).abs
    Real.1 * (u.sin - v.sin).abs = (u.sin - v.sin).abs
    v.cos.abs * (u.sin - v.sin).abs <= (u.sin - v.sin).abs
    lte_trans(v.cos.abs * (u.sin - v.sin).abs, (u.sin - v.sin).abs, (u - v).abs)
    v.cos.abs * (u.sin - v.sin).abs <= (u - v).abs
    add_le_add(u.sin.abs * (u.cos - v.cos).abs, (u - v).abs, v.cos.abs * (u.sin - v.sin).abs, (u - v).abs)
    u.sin.abs * (u.cos - v.cos).abs + v.cos.abs * (u.sin - v.sin).abs <= (u - v).abs + (u - v).abs
    (u - v).abs + (u - v).abs = two * (u - v).abs
    u.sin.abs * (u.cos - v.cos).abs + v.cos.abs * (u.sin - v.sin).abs <= two * (u - v).abs
    lte_trans((u.sin * (u.cos - v.cos) + v.cos * (u.sin - v.sin)).abs, u.sin.abs * (u.cos - v.cos).abs + v.cos.abs * (u.sin - v.sin).abs, two * (u - v).abs)
    (u.sin * (u.cos - v.cos) + v.cos * (u.sin - v.sin)).abs <= two * (u - v).abs
    (u.sin * u.cos - v.sin * v.cos).abs <= two * (u - v).abs
    (sincos(u) - sincos(v)).abs <= two * (u - v).abs
}

/// The product Real.sin * Real.cos is continuous.
theorem sincos_continuous {
    continuous(sincos)
} by {
    sin_continuous
    continuous(Real.sin)
    cos_continuous
    continuous(Real.cos)
    continuous_pointwise_mul(Real.sin, Real.cos)
    continuous(pointwise_mul[Real, Real](Real.sin, Real.cos))
    forall(x: Real) {
        pointwise_mul[Real, Real](Real.sin, Real.cos)(x) = x.sin * x.cos
        sincos(x) = x.sin * x.cos
        pointwise_mul[Real, Real](Real.sin, Real.cos)(x) = sincos(x)
    }
    function_extensionality(pointwise_mul[Real, Real](Real.sin, Real.cos), sincos)
    pointwise_mul[Real, Real](Real.sin, Real.cos) = sincos
    continuous(sincos)
}

// ---------------------------------------------------------------------------
// The integral of Real.sin * Real.cos over [0, 2*pi] is zero: the first genuine
// orthogonality instance among the harmonics.  The antiderivative of
// Real.sin * Real.cos is Real.sin^2 / 2 (by the product rule and Real.sin' = Real.cos), and the
// fundamental theorem gives integral(Real.sin Real.cos, 0, 2*pi) = (2*pi).sin^2/2 -
// (0).sin^2/2 = 0.
// ---------------------------------------------------------------------------

/// The pointwise function x -> x.sin * x.sin is the square of sine.
define sin_sq_fn(x: Real) -> Real {
    x.sin * x.sin
}

/// x.sin^2 / 2, the antiderivative of Real.sin * Real.cos.
define sincos_antiderivative(x: Real) -> Real {
    Real.one_half * sin_sq_fn(x)
}

/// The pointwise derivative of Real.sin^2 is 2 Real.sin Real.cos.
theorem sin_sq_derivative {
    is_derivative_fn(sin_sq_fn, pointwise_mul(constant[Real, Real](two), sincos))
} by {
    sin_is_derivative_fn
    is_derivative_fn(Real.sin, Real.cos)
    derivative_fn_square(Real.sin, Real.cos)
    is_derivative_fn(pointwise_mul(Real.sin, Real.sin), pointwise_add(pointwise_mul(Real.sin, Real.cos), pointwise_mul(Real.sin, Real.cos)))
    forall(x: Real) {
        pointwise_mul(Real.sin, Real.sin)(x) = x.sin * x.sin
        sin_sq_fn(x) = x.sin * x.sin
        pointwise_mul(Real.sin, Real.sin)(x) = sin_sq_fn(x)
        pointwise_add_apply(pointwise_mul(Real.sin, Real.cos), pointwise_mul(Real.sin, Real.cos), x)
        pointwise_add(pointwise_mul(Real.sin, Real.cos), pointwise_mul(Real.sin, Real.cos))(x) =
            pointwise_mul(Real.sin, Real.cos)(x) + pointwise_mul(Real.sin, Real.cos)(x)
        pointwise_mul_apply(Real.sin, Real.cos, x)
        pointwise_mul(Real.sin, Real.cos)(x) = x.sin * x.cos
        sincos(x) = x.sin * x.cos
        pointwise_mul(Real.sin, Real.cos)(x) = sincos(x)
        pointwise_mul(Real.sin, Real.cos)(x) + pointwise_mul(Real.sin, Real.cos)(x) = sincos(x) + sincos(x)
        sincos(x) + sincos(x) = two * sincos(x)
        pointwise_mul_apply(constant[Real, Real](two), sincos, x)
        pointwise_mul(constant[Real, Real](two), sincos)(x) = constant[Real, Real](two, x) * sincos(x)
        constant[Real, Real](two, x) = two
        pointwise_mul(constant[Real, Real](two), sincos)(x) = two * sincos(x)
        pointwise_add(pointwise_mul(Real.sin, Real.cos), pointwise_mul(Real.sin, Real.cos))(x) =
            pointwise_mul(constant[Real, Real](two), sincos)(x)
    }
    function_extensionality(pointwise_mul(Real.sin, Real.sin), sin_sq_fn)
    pointwise_mul(Real.sin, Real.sin) = sin_sq_fn
    function_extensionality(
        pointwise_add(pointwise_mul(Real.sin, Real.cos), pointwise_mul(Real.sin, Real.cos)),
        pointwise_mul(constant[Real, Real](two), sincos))
    pointwise_add(pointwise_mul(Real.sin, Real.cos), pointwise_mul(Real.sin, Real.cos)) =
        pointwise_mul(constant[Real, Real](two), sincos)
    is_derivative_fn(sin_sq_fn, pointwise_mul(constant[Real, Real](two), sincos))
}

/// The pointwise derivative of Real.sin^2 / 2 is Real.sin * Real.cos.
theorem sincos_antiderivative_derivative {
    is_derivative_fn(sincos_antiderivative, sincos)
} by {
    sin_sq_derivative
    is_derivative_fn(sin_sq_fn, pointwise_mul(constant[Real, Real](two), sincos))
    derivative_fn_const_mul(Real.one_half, sin_sq_fn, pointwise_mul(constant[Real, Real](two), sincos))
    is_derivative_fn(
        pointwise_mul(constant[Real, Real](Real.one_half), sin_sq_fn),
        pointwise_mul(constant[Real, Real](Real.one_half), pointwise_mul(constant[Real, Real](two), sincos)))
    forall(x: Real) {
        pointwise_mul_apply(constant[Real, Real](Real.one_half), sin_sq_fn, x)
        pointwise_mul(constant[Real, Real](Real.one_half), sin_sq_fn)(x) =
            constant[Real, Real](Real.one_half, x) * sin_sq_fn(x)
        constant[Real, Real](Real.one_half, x) = Real.one_half
        pointwise_mul(constant[Real, Real](Real.one_half), sin_sq_fn)(x) = Real.one_half * sin_sq_fn(x)
        sincos_antiderivative(x) = Real.one_half * sin_sq_fn(x)
        pointwise_mul(constant[Real, Real](Real.one_half), sin_sq_fn)(x) = sincos_antiderivative(x)
    }
    function_extensionality(
        pointwise_mul(constant[Real, Real](Real.one_half), sin_sq_fn), sincos_antiderivative)
    pointwise_mul(constant[Real, Real](Real.one_half), sin_sq_fn) = sincos_antiderivative
    forall(x: Real) {
        pointwise_mul_apply(constant[Real, Real](Real.one_half), pointwise_mul(constant[Real, Real](two), sincos), x)
        pointwise_mul(constant[Real, Real](Real.one_half), pointwise_mul(constant[Real, Real](two), sincos))(x) =
            constant[Real, Real](Real.one_half, x) * pointwise_mul(constant[Real, Real](two), sincos)(x)
        pointwise_mul_apply(constant[Real, Real](two), sincos, x)
        pointwise_mul(constant[Real, Real](two), sincos)(x) = constant[Real, Real](two, x) * sincos(x)
        pointwise_mul(constant[Real, Real](Real.one_half), pointwise_mul(constant[Real, Real](two), sincos))(x) =
            Real.one_half * (two * sincos(x))
        one_half_plus_one_half
        Real.one_half + Real.one_half = Real.1
        Real.one_half * two = Real.1
        mul_assoc(Real.one_half, two, sincos(x))
        Real.one_half * (two * sincos(x)) = (Real.one_half * two) * sincos(x)
        (Real.one_half * two) * sincos(x) = Real.1 * sincos(x)
        Real.1 * sincos(x) = sincos(x)
        Real.one_half * (two * sincos(x)) = sincos(x)
        pointwise_mul(constant[Real, Real](Real.one_half), pointwise_mul(constant[Real, Real](two), sincos))(x) =
            sincos(x)
    }
    function_extensionality(
        pointwise_mul(constant[Real, Real](Real.one_half), pointwise_mul(constant[Real, Real](two), sincos)),
        sincos)
    pointwise_mul(constant[Real, Real](Real.one_half), pointwise_mul(constant[Real, Real](two), sincos)) =
        sincos
    is_derivative_fn(sincos_antiderivative, sincos)
}

/// The antiderivative Real.sin^2 / 2 of Real.sin * Real.cos is continuous.
theorem sincos_antiderivative_continuous {
    continuous(sincos_antiderivative)
} by {
    sin_continuous
    continuous(Real.sin)
    continuous_pointwise_mul(Real.sin, Real.sin)
    continuous(pointwise_mul[Real, Real](Real.sin, Real.sin))
    forall(x: Real) {
        pointwise_mul[Real, Real](Real.sin, Real.sin)(x) = x.sin * x.sin
        sin_sq_fn(x) = x.sin * x.sin
        pointwise_mul[Real, Real](Real.sin, Real.sin)(x) = sin_sq_fn(x)
    }
    function_extensionality(pointwise_mul[Real, Real](Real.sin, Real.sin), sin_sq_fn)
    pointwise_mul[Real, Real](Real.sin, Real.sin) = sin_sq_fn
    continuous(sin_sq_fn)
    continuous_const_mul_left(Real.one_half, sin_sq_fn)
    continuous(sin_sq_fn) implies continuous(const_mul_left(Real.one_half, sin_sq_fn))
    continuous(const_mul_left(Real.one_half, sin_sq_fn))
    forall(x: Real) {
        const_mul_left(Real.one_half, sin_sq_fn)(x) = Real.one_half * sin_sq_fn(x)
        sincos_antiderivative(x) = Real.one_half * sin_sq_fn(x)
        const_mul_left(Real.one_half, sin_sq_fn)(x) = sincos_antiderivative(x)
    }
    function_extensionality(const_mul_left(Real.one_half, sin_sq_fn), sincos_antiderivative)
    const_mul_left(Real.one_half, sin_sq_fn) = sincos_antiderivative
    continuous(sincos_antiderivative)
}

/// Real.sin * Real.cos is integrable on [0, 2*pi].
theorem sincos_integrable_zero_two_pi {
    is_integrable(sincos, Real.0, two * pi)
} by {
    two_pi_pos
    Real.0 < two * pi
    lt_imp_lte(Real.0, two * pi)
    Real.0 <= two * pi
    two_positive
    two.is_positive
    pos_gt_zero(two)
    two.is_positive implies two > Real.0
    two > Real.0
    lt_imp_lte(Real.0, two)
    Real.0 <= two
    sincos_antiderivative_continuous
    continuous(sincos_antiderivative)
    sincos_antiderivative_derivative
    is_derivative_fn(sincos_antiderivative, sincos)
    forall(u: Real, v: Real) {
        if interval_contains(Real.0, two * pi, u) and interval_contains(Real.0, two * pi, v) {
            sincos_lipschitz_m2(u, v)
            (sincos(u) - sincos(v)).abs <= two * (u - v).abs
        }
    }
    forall(t: Real) {
        if interval_contains(Real.0, two * pi, t) {
            sincos_ge_neg_one(t)
            -Real.1 <= sincos(t)
        }
    }
    forall(t: Real) {
        if interval_contains(Real.0, two * pi, t) {
            sincos_le_one(t)
            sincos(t) <= Real.1
        }
    }
    Real.0 <= two * pi
    (Real.0 <= two * pi) and Real.0 <= two
    ((Real.0 <= two * pi) and Real.0 <= two) and continuous(sincos_antiderivative)
    (((Real.0 <= two * pi) and Real.0 <= two) and continuous(sincos_antiderivative)) and is_derivative_fn(sincos_antiderivative, sincos)
    ((((Real.0 <= two * pi) and Real.0 <= two) and continuous(sincos_antiderivative)) and is_derivative_fn(sincos_antiderivative, sincos)) and forall(u: Real, v: Real) {
        interval_contains(Real.0, two * pi, u) and interval_contains(Real.0, two * pi, v)
        implies (sincos(u) - sincos(v)).abs <= two * (u - v).abs
    }
    (((((Real.0 <= two * pi) and Real.0 <= two) and continuous(sincos_antiderivative)) and is_derivative_fn(sincos_antiderivative, sincos)) and forall(u: Real, v: Real) {
        interval_contains(Real.0, two * pi, u) and interval_contains(Real.0, two * pi, v)
        implies (sincos(u) - sincos(v)).abs <= two * (u - v).abs
    }) and forall(t: Real) { interval_contains(Real.0, two * pi, t) implies -Real.1 <= sincos(t) }
    ((((((Real.0 <= two * pi) and Real.0 <= two) and continuous(sincos_antiderivative)) and is_derivative_fn(sincos_antiderivative, sincos)) and forall(u: Real, v: Real) {
        interval_contains(Real.0, two * pi, u) and interval_contains(Real.0, two * pi, v)
        implies (sincos(u) - sincos(v)).abs <= two * (u - v).abs
    }) and forall(t: Real) { interval_contains(Real.0, two * pi, t) implies -Real.1 <= sincos(t) }) and forall(t: Real) { interval_contains(Real.0, two * pi, t) implies sincos(t) <= Real.1 }
    if ((((((Real.0 <= two * pi) and Real.0 <= two) and continuous(sincos_antiderivative)) and is_derivative_fn(sincos_antiderivative, sincos)) and forall(u: Real, v: Real) {
        interval_contains(Real.0, two * pi, u) and interval_contains(Real.0, two * pi, v)
        implies (sincos(u) - sincos(v)).abs <= two * (u - v).abs
    }) and forall(t: Real) { interval_contains(Real.0, two * pi, t) implies -Real.1 <= sincos(t) }) and forall(t: Real) { interval_contains(Real.0, two * pi, t) implies sincos(t) <= Real.1 } {
        fn_integrable_gen(sincos, sincos_antiderivative, Real.0, two * pi, two, -Real.1, Real.1)
        is_integrable(sincos, Real.0, two * pi)
    }
}

/// The integral of Real.sin * Real.cos over [0, 2*pi] is zero: the orthogonality of
/// the harmonics Real.sin and Real.cos on a full period.
theorem integral_sincos_zero_two_pi {
    integral(sincos, Real.0, two * pi) = Real.0
} by {
    sincos_integrable_zero_two_pi
    is_integrable(sincos, Real.0, two * pi)
    two_pi_pos
    Real.0 < two * pi
    lt_imp_lte(Real.0, two * pi)
    Real.0 <= two * pi
    sincos_antiderivative_continuous
    continuous(sincos_antiderivative)
    sincos_antiderivative_derivative
    is_derivative_fn(sincos_antiderivative, sincos)
    forall(t: Real) {
        if interval_contains(Real.0, two * pi, t) {
            sincos_ge_neg_one(t)
            -Real.1 <= sincos(t)
        }
    }
    forall(t: Real) {
        if interval_contains(Real.0, two * pi, t) {
            sincos_le_one(t)
            sincos(t) <= Real.1
        }
    }
    Real.0 <= two * pi
    (Real.0 <= two * pi) and continuous(sincos_antiderivative)
    ((Real.0 <= two * pi) and continuous(sincos_antiderivative)) and is_derivative_fn(sincos_antiderivative, sincos)
    (((Real.0 <= two * pi) and continuous(sincos_antiderivative)) and is_derivative_fn(sincos_antiderivative, sincos)) and is_integrable(sincos, Real.0, two * pi)
    ((((Real.0 <= two * pi) and continuous(sincos_antiderivative)) and is_derivative_fn(sincos_antiderivative, sincos)) and is_integrable(sincos, Real.0, two * pi)) and forall(t: Real) { interval_contains(Real.0, two * pi, t) implies -Real.1 <= sincos(t) }
    (((((Real.0 <= two * pi) and continuous(sincos_antiderivative)) and is_derivative_fn(sincos_antiderivative, sincos)) and is_integrable(sincos, Real.0, two * pi)) and forall(t: Real) { interval_contains(Real.0, two * pi, t) implies -Real.1 <= sincos(t) }) and forall(t: Real) { interval_contains(Real.0, two * pi, t) implies sincos(t) <= Real.1 }
    if (((((Real.0 <= two * pi) and continuous(sincos_antiderivative)) and is_derivative_fn(sincos_antiderivative, sincos)) and is_integrable(sincos, Real.0, two * pi)) and forall(t: Real) { interval_contains(Real.0, two * pi, t) implies -Real.1 <= sincos(t) }) and forall(t: Real) { interval_contains(Real.0, two * pi, t) implies sincos(t) <= Real.1 } {
        ftc2_general(sincos, sincos_antiderivative, Real.0, two * pi, -Real.1, Real.1)
        integral(sincos, Real.0, two * pi) = sincos_antiderivative(two * pi) - sincos_antiderivative(Real.0)
        sincos_antiderivative(two * pi) = Real.one_half * sin_sq_fn(two * pi)
        sin_sq_fn(two * pi) = (two * pi).sin * (two * pi).sin
        sin_two_pi_value
        (two * pi).sin = Real.0
        (two * pi).sin * (two * pi).sin = Real.0
        sin_sq_fn(two * pi) = Real.0
        Real.one_half * sin_sq_fn(two * pi) = Real.0
        sincos_antiderivative(two * pi) = Real.0
        sincos_antiderivative(Real.0) = Real.one_half * sin_sq_fn(Real.0)
        sin_sq_fn(Real.0) = (Real.0).sin * (Real.0).sin
        sin_zero
        (Real.0).sin = Real.0
        (Real.0).sin * (Real.0).sin = Real.0
        sin_sq_fn(Real.0) = Real.0
        Real.one_half * sin_sq_fn(Real.0) = Real.0
        sincos_antiderivative(Real.0) = Real.0
        sincos_antiderivative(two * pi) - sincos_antiderivative(Real.0) = Real.0 - Real.0
        Real.0 - Real.0 = Real.0
        integral(sincos, Real.0, two * pi) = Real.0
    }
}

// ---------------------------------------------------------------------------
// The Dirichlet kernel: the summation identity
//   2 (x/2).sin * (1/2 + sum_{k=1}^{n} (k x).cos) = ((n + 1/2) x).sin
// for the trigonometric polynomials of a partial Fourier sum.  This is the
// classic trig-polynomial summation identity, proved here with the exported
// sine/cosine API only.
// ---------------------------------------------------------------------------

/// The k-th cosine harmonic (k x).cos.
define cos_harmonic(x: Real, k: Nat) -> Real {
    (from_nat[Real](k) * x).cos
}

/// The sum of the first n cosine harmonics, sum_{k=1}^{n} (k x).cos,
/// indexed as a partial sum over the shifted sequence so that k runs from
/// one to n.
define cos_harmonic_sum(x: Real, n: Nat) -> Real {
    partial(compose(cos_harmonic(x), Nat.suc), n)
}

/// The half-angle harmonic ((k + 1/2) x).sin.
define half_angle_harmonic(x: Real, k: Nat) -> Real {
    ((from_nat[Real](k) + Real.one_half) * x).sin
}

/// The successive difference of the half-angle harmonics at index i:
/// a(i + 1) - a(i) with a(i) = ((i + 1/2) x).sin.
define half_angle_diff(x: Real, i: Nat) -> Real {
    half_angle_harmonic(x, i.suc) - half_angle_harmonic(x, i)
}

/// The partial sums of the successive differences telescope:
/// sum_{i=0}^{n-1} (a(i+1) - a(i)) = a(n) - a(0).
theorem half_angle_diff_telescopes(x: Real, n: Nat) {
    partial(half_angle_diff(x), n) = half_angle_harmonic(x, n) - half_angle_harmonic(x, Nat.0)
} by {
    define q(m: Nat) -> Bool {
        partial(half_angle_diff(x), m) = half_angle_harmonic(x, m) - half_angle_harmonic(x, Nat.0)
    }
    partial(half_angle_diff(x), Nat.0) = Real.0
    half_angle_harmonic(x, Nat.0) - half_angle_harmonic(x, Nat.0) = Real.0
    partial(half_angle_diff(x), Nat.0) = half_angle_harmonic(x, Nat.0) - half_angle_harmonic(x, Nat.0)
    q(Nat.0)
    forall(m: Nat) {
        if q(m) {
            partial(half_angle_diff(x), m.suc) = partial(half_angle_diff(x), m) + half_angle_diff(x, m)
            half_angle_diff(x, m) = half_angle_harmonic(x, m.suc) - half_angle_harmonic(x, m)
            partial(half_angle_diff(x), m) + half_angle_diff(x, m) =
                half_angle_harmonic(x, m) - half_angle_harmonic(x, Nat.0) +
                (half_angle_harmonic(x, m.suc) - half_angle_harmonic(x, m))
            half_angle_harmonic(x, m) - half_angle_harmonic(x, Nat.0) +
                (half_angle_harmonic(x, m.suc) - half_angle_harmonic(x, m)) =
                half_angle_harmonic(x, m.suc) - half_angle_harmonic(x, Nat.0)
            partial(half_angle_diff(x), m.suc) = half_angle_harmonic(x, m.suc) - half_angle_harmonic(x, Nat.0)
            q(m.suc)
        }
    }
    alt_induction(q)
    (q(Nat.0) and forall(k: Nat) { q(k) implies q(k.suc) }) implies forall(k: Nat) { q(k) }
    q(Nat.0) and forall(k: Nat) { q(k) implies q(k.suc) }
    forall(k: Nat) { q(k) }
    q(n)
}

/// 2 (x/2).sin ((i+1) x).cos = ((i + 3/2) x).sin - ((i + 1/2) x).sin.
theorem half_angle_diff_expansion(x: Real, i: Nat) {
    two * (x * Real.one_half).sin * cos_harmonic(x, i.suc) = half_angle_diff(x, i)
} by {
    half_angle_diff(x, i) = half_angle_harmonic(x, i.suc) - half_angle_harmonic(x, i)
    half_angle_harmonic(x, i.suc) = ((from_nat[Real](i.suc) + Real.one_half) * x).sin
    half_angle_harmonic(x, i) = ((from_nat[Real](i) + Real.one_half) * x).sin
    (from_nat[Real](i.suc) + Real.one_half) * x = from_nat[Real](i.suc) * x + Real.one_half * x
    from_nat_add[Real](i, Nat.1)
    from_nat[Real](i + Nat.1) = from_nat[Real](i) + from_nat[Real](Nat.1)
    i + Nat.1 = i.suc
    from_nat[Real](i.suc) = from_nat[Real](i) + from_nat[Real](Nat.1)
    from_nat[Real](Nat.1) = Real.1
    from_nat[Real](i.suc) = from_nat[Real](i) + Real.1
    one_half_plus_one_half
    Real.one_half + Real.one_half = Real.1
    sub_moves_sides(Real.one_half, Real.one_half, Real.1)
    Real.one_half = Real.1 - Real.one_half
    from_nat[Real](i) + Real.one_half = from_nat[Real](i) + (Real.1 - Real.one_half)
    from_nat[Real](i) + (Real.1 - Real.one_half) = from_nat[Real](i) + Real.1 - Real.one_half
    from_nat[Real](i) + Real.one_half = from_nat[Real](i.suc) - Real.one_half
    mul_sub_distrib_left(from_nat[Real](i.suc), Real.one_half, x)
    (from_nat[Real](i.suc) - Real.one_half) * x = from_nat[Real](i.suc) * x - Real.one_half * x
    (from_nat[Real](i) + Real.one_half) * x = (from_nat[Real](i.suc) - Real.one_half) * x
    (from_nat[Real](i) + Real.one_half) * x = from_nat[Real](i.suc) * x - Real.one_half * x
    half_angle_harmonic(x, i.suc) = (from_nat[Real](i.suc) * x + Real.one_half * x).sin
    half_angle_harmonic(x, i) = (from_nat[Real](i.suc) * x - Real.one_half * x).sin
    half_angle_diff(x, i) =
        (from_nat[Real](i.suc) * x + Real.one_half * x).sin - (from_nat[Real](i.suc) * x - Real.one_half * x).sin
    sin_add(from_nat[Real](i.suc) * x, Real.one_half * x)
    (from_nat[Real](i.suc) * x + Real.one_half * x).sin =
        (from_nat[Real](i.suc) * x).sin * (Real.one_half * x).cos + (from_nat[Real](i.suc) * x).cos * (Real.one_half * x).sin
    sin_add(from_nat[Real](i.suc) * x, -(Real.one_half * x))
    (from_nat[Real](i.suc) * x + (-(Real.one_half * x))).sin =
        (from_nat[Real](i.suc) * x).sin * (-(Real.one_half * x)).cos + (from_nat[Real](i.suc) * x).cos * (-(Real.one_half * x)).sin
    from_nat[Real](i.suc) * x + (-(Real.one_half * x)) = from_nat[Real](i.suc) * x - Real.one_half * x
    (from_nat[Real](i.suc) * x - Real.one_half * x).sin =
        (from_nat[Real](i.suc) * x).sin * (-(Real.one_half * x)).cos + (from_nat[Real](i.suc) * x).cos * (-(Real.one_half * x)).sin
    cos_neg(Real.one_half * x)
    (-(Real.one_half * x)).cos = (Real.one_half * x).cos
    sin_neg(Real.one_half * x)
    (-(Real.one_half * x)).sin = -(Real.one_half * x).sin
    (from_nat[Real](i.suc) * x).sin * (-(Real.one_half * x)).cos = (from_nat[Real](i.suc) * x).sin * (Real.one_half * x).cos
    (from_nat[Real](i.suc) * x).cos * (-(Real.one_half * x)).sin = (from_nat[Real](i.suc) * x).cos * (-(Real.one_half * x).sin)
    mul_neg_one_right((Real.one_half * x).sin)
    (Real.one_half * x).sin * (-Real.1) = -(Real.one_half * x).sin
    (from_nat[Real](i.suc) * x).cos * (-(Real.one_half * x).sin) = (from_nat[Real](i.suc) * x).cos * ((Real.one_half * x).sin * (-Real.1))
    mul_assoc((from_nat[Real](i.suc) * x).cos, (Real.one_half * x).sin, -Real.1)
    (from_nat[Real](i.suc) * x).cos * ((Real.one_half * x).sin * (-Real.1)) = ((from_nat[Real](i.suc) * x).cos * (Real.one_half * x).sin) * (-Real.1)
    mul_neg_one_right((from_nat[Real](i.suc) * x).cos * (Real.one_half * x).sin)
    ((from_nat[Real](i.suc) * x).cos * (Real.one_half * x).sin) * (-Real.1) = -((from_nat[Real](i.suc) * x).cos * (Real.one_half * x).sin)
    (from_nat[Real](i.suc) * x).cos * (-(Real.one_half * x)).sin = -((from_nat[Real](i.suc) * x).cos * (Real.one_half * x).sin)
    (from_nat[Real](i.suc) * x).sin * (-(Real.one_half * x)).cos + (from_nat[Real](i.suc) * x).cos * (-(Real.one_half * x)).sin =
        (from_nat[Real](i.suc) * x).sin * (Real.one_half * x).cos + (from_nat[Real](i.suc) * x).cos * (-(Real.one_half * x)).sin
    (from_nat[Real](i.suc) * x).sin * (Real.one_half * x).cos + (from_nat[Real](i.suc) * x).cos * (-(Real.one_half * x)).sin =
        (from_nat[Real](i.suc) * x).sin * (Real.one_half * x).cos + (-((from_nat[Real](i.suc) * x).cos * (Real.one_half * x).sin))
    (from_nat[Real](i.suc) * x).sin * (Real.one_half * x).cos + (-((from_nat[Real](i.suc) * x).cos * (Real.one_half * x).sin)) =
        (from_nat[Real](i.suc) * x).sin * (Real.one_half * x).cos - (from_nat[Real](i.suc) * x).cos * (Real.one_half * x).sin
    (from_nat[Real](i.suc) * x).sin * (-(Real.one_half * x)).cos + (from_nat[Real](i.suc) * x).cos * (-(Real.one_half * x)).sin =
        (from_nat[Real](i.suc) * x).sin * (Real.one_half * x).cos - (from_nat[Real](i.suc) * x).cos * (Real.one_half * x).sin
    (from_nat[Real](i.suc) * x - Real.one_half * x).sin =
        (from_nat[Real](i.suc) * x).sin * (Real.one_half * x).cos - (from_nat[Real](i.suc) * x).cos * (Real.one_half * x).sin
    (from_nat[Real](i.suc) * x + Real.one_half * x).sin - (from_nat[Real](i.suc) * x - Real.one_half * x).sin =
        ((from_nat[Real](i.suc) * x).sin * (Real.one_half * x).cos + (from_nat[Real](i.suc) * x).cos * (Real.one_half * x).sin) -
        ((from_nat[Real](i.suc) * x).sin * (Real.one_half * x).cos - (from_nat[Real](i.suc) * x).cos * (Real.one_half * x).sin)
    ring_sub_sub_comm((from_nat[Real](i.suc) * x).sin * (Real.one_half * x).cos, (from_nat[Real](i.suc) * x).cos * (Real.one_half * x).sin)
    ((from_nat[Real](i.suc) * x).sin * (Real.one_half * x).cos + (from_nat[Real](i.suc) * x).cos * (Real.one_half * x).sin) -
        ((from_nat[Real](i.suc) * x).sin * (Real.one_half * x).cos - (from_nat[Real](i.suc) * x).cos * (Real.one_half * x).sin) =
        (from_nat[Real](i.suc) * x).cos * (Real.one_half * x).sin + (from_nat[Real](i.suc) * x).cos * (Real.one_half * x).sin
    (from_nat[Real](i.suc) * x).cos * (Real.one_half * x).sin + (from_nat[Real](i.suc) * x).cos * (Real.one_half * x).sin =
        two * ((from_nat[Real](i.suc) * x).cos * (Real.one_half * x).sin)
    mul_assoc(two, (from_nat[Real](i.suc) * x).cos, (Real.one_half * x).sin)
    two * ((from_nat[Real](i.suc) * x).cos * (Real.one_half * x).sin) = two * (from_nat[Real](i.suc) * x).cos * (Real.one_half * x).sin
    (from_nat[Real](i.suc) * x + Real.one_half * x).sin - (from_nat[Real](i.suc) * x - Real.one_half * x).sin =
        two * (from_nat[Real](i.suc) * x).cos * (Real.one_half * x).sin
    half_angle_diff(x, i) = two * (from_nat[Real](i.suc) * x).cos * (Real.one_half * x).sin
    cos_harmonic(x, i.suc) = (from_nat[Real](i.suc) * x).cos
    two * (from_nat[Real](i.suc) * x).cos * (Real.one_half * x).sin =
        two * (x * Real.one_half).sin * cos_harmonic(x, i.suc)
    two * (x * Real.one_half).sin * cos_harmonic(x, i.suc) = half_angle_diff(x, i)
}

/// The Dirichlet-kernel identity:
/// 2 (x/2).sin * (1/2 + sum_{k=1}^{n} (k x).cos) = ((n + 1/2) x).sin.
theorem dirichlet_kernel_identity(x: Real, n: Nat) {
    two * (x * Real.one_half).sin * (Real.one_half + cos_harmonic_sum(x, n)) = half_angle_harmonic(x, n)
} by {
    cos_harmonic_sum(x, n) = partial(compose(cos_harmonic(x), Nat.suc), n)
    partial_scalar_mul(two * (x * Real.one_half).sin, compose(cos_harmonic(x), Nat.suc), n)
    (two * (x * Real.one_half).sin) * partial(compose(cos_harmonic(x), Nat.suc), n) =
        partial(mul_fn(two * (x * Real.one_half).sin, compose(cos_harmonic(x), Nat.suc)), n)
    forall(i: Nat) {
        if i < n {
            half_angle_diff_expansion(x, i)
            two * (x * Real.one_half).sin * cos_harmonic(x, i.suc) = half_angle_diff(x, i)
            mul_fn(two * (x * Real.one_half).sin, compose(cos_harmonic(x), Nat.suc))(i) =
                (two * (x * Real.one_half).sin) * compose(cos_harmonic(x), Nat.suc)(i)
            compose(cos_harmonic(x), Nat.suc)(i) = cos_harmonic(x, i.suc)
            (two * (x * Real.one_half).sin) * compose(cos_harmonic(x), Nat.suc)(i) =
                two * (x * Real.one_half).sin * cos_harmonic(x, i.suc)
            mul_fn(two * (x * Real.one_half).sin, compose(cos_harmonic(x), Nat.suc))(i) = half_angle_diff(x, i)
        }
    }
    partial_pointwise_eq(mul_fn(two * (x * Real.one_half).sin, compose(cos_harmonic(x), Nat.suc)), half_angle_diff(x), n)
    partial(mul_fn(two * (x * Real.one_half).sin, compose(cos_harmonic(x), Nat.suc)), n) = partial(half_angle_diff(x), n)
    half_angle_diff_telescopes(x, n)
    partial(half_angle_diff(x), n) = half_angle_harmonic(x, n) - half_angle_harmonic(x, Nat.0)
    partial(mul_fn(two * (x * Real.one_half).sin, compose(cos_harmonic(x), Nat.suc)), n) =
        half_angle_harmonic(x, n) - half_angle_harmonic(x, Nat.0)
    (two * (x * Real.one_half).sin) * partial(compose(cos_harmonic(x), Nat.suc), n) =
        half_angle_harmonic(x, n) - half_angle_harmonic(x, Nat.0)
    two * (x * Real.one_half).sin * cos_harmonic_sum(x, n) = half_angle_harmonic(x, n) - half_angle_harmonic(x, Nat.0)
    two * (x * Real.one_half).sin * (Real.one_half + cos_harmonic_sum(x, n)) =
        two * (x * Real.one_half).sin * Real.one_half + two * (x * Real.one_half).sin * cos_harmonic_sum(x, n)
    one_half_plus_one_half
    Real.one_half + Real.one_half = Real.1
    Real.one_half * two = Real.1
    mul_assoc(two, (x * Real.one_half).sin, Real.one_half)
    two * (x * Real.one_half).sin * Real.one_half = two * ((x * Real.one_half).sin * Real.one_half)
    real_mul_comm((x * Real.one_half).sin, Real.one_half)
    (x * Real.one_half).sin * Real.one_half = Real.one_half * (x * Real.one_half).sin
    two * ((x * Real.one_half).sin * Real.one_half) = two * (Real.one_half * (x * Real.one_half).sin)
    mul_assoc(two, Real.one_half, (x * Real.one_half).sin)
    two * (Real.one_half * (x * Real.one_half).sin) = (two * Real.one_half) * (x * Real.one_half).sin
    real_mul_comm(two, Real.one_half)
    two * Real.one_half = Real.one_half * two
    (two * Real.one_half) * (x * Real.one_half).sin = (Real.one_half * two) * (x * Real.one_half).sin
    (Real.one_half * two) * (x * Real.one_half).sin = Real.1 * (x * Real.one_half).sin
    Real.1 * (x * Real.one_half).sin = (x * Real.one_half).sin
    two * (x * Real.one_half).sin * Real.one_half = (x * Real.one_half).sin
    two * (x * Real.one_half).sin * (Real.one_half + cos_harmonic_sum(x, n)) =
        (x * Real.one_half).sin + (half_angle_harmonic(x, n) - half_angle_harmonic(x, Nat.0))
    half_angle_harmonic(x, Nat.0) = ((from_nat[Real](Nat.0) + Real.one_half) * x).sin
    from_nat[Real](Nat.0) = Real.0
    (from_nat[Real](Nat.0) + Real.one_half) * x = Real.one_half * x
    half_angle_harmonic(x, Nat.0) = (Real.one_half * x).sin
    (Real.one_half * x).sin = (x * Real.one_half).sin
    half_angle_harmonic(x, Nat.0) = (x * Real.one_half).sin
    (x * Real.one_half).sin + (half_angle_harmonic(x, n) - half_angle_harmonic(x, Nat.0)) =
        (x * Real.one_half).sin + (half_angle_harmonic(x, n) - (x * Real.one_half).sin)
    add_comm((x * Real.one_half).sin, half_angle_harmonic(x, n) - (x * Real.one_half).sin)
    (x * Real.one_half).sin + (half_angle_harmonic(x, n) - (x * Real.one_half).sin) =
        (half_angle_harmonic(x, n) - (x * Real.one_half).sin) + (x * Real.one_half).sin
    sub_add_cancel(half_angle_harmonic(x, n), (x * Real.one_half).sin)
    (half_angle_harmonic(x, n) - (x * Real.one_half).sin) + (x * Real.one_half).sin = half_angle_harmonic(x, n)
    (x * Real.one_half).sin + (half_angle_harmonic(x, n) - half_angle_harmonic(x, Nat.0)) =
        half_angle_harmonic(x, n)
    two * (x * Real.one_half).sin * (Real.one_half + cos_harmonic_sum(x, n)) = half_angle_harmonic(x, n)
}
