/// Forward differences of functions on a multiplicative type.

from algebra.add_comm_group import AddCommGroup
from algebra.add_group import inverse_add
from data.basic.function_algebra import pointwise_add
from algebra.module.module import Module, module_smul_add_right, module_smul_neg_right
from algebra.mul import Mul
from algebra.ring.ring import Ring

/// The forward shift of a function by left multiplication with a fixed element.
define fwd_shift[G: Mul, A](g: G, f: G -> A, x: G) -> A {
    f(g * x)
}

/// The forward difference of a function by left multiplication with a fixed element.
define fwd_diff[G: Mul, A: AddCommGroup](g: G, f: G -> A, x: G) -> A {
    f(g * x) - f(x)
}

/// Pointwise scalar multiplication of functions valued in a module.
define pointwise_smul[R: Ring, A: AddCommGroup, G](act: Module[R, A], r: R, f: G -> A, x: G) -> A {
    act.smul(r, f(x))
}

/// The forward shift agrees with left multiplication on the input.
theorem fwd_shift_apply[G: Mul, A](g: G, f: G -> A, x: G) {
    fwd_shift(g, f, x) = f(g * x)
}

/// The forward difference is the shifted value minus the original value.
theorem fwd_diff_apply[G: Mul, A: AddCommGroup](g: G, f: G -> A, x: G) {
    fwd_diff(g, f, x) = f(g * x) - f(x)
}

/// Subtracting two sums is the sum of the corresponding differences.
theorem add_sub_add_eq_sub_add_sub[A: AddCommGroup](a: A, b: A, c: A, d: A) {
    (a + b) - (c + d) = (a - c) + (b - d)
} by {
    inverse_add(c, d)
    -(c + d) = -d + -c
    -d + -c = -c + -d
    -(c + d) = -c + -d
    (a + b) - (c + d) = a + b + -c + -d
    a - c = a + -c
    b - d = b + -d
    (a - c) + (b - d) = a + -c + b + -d
    -c + b = b + -c
    a + -c + b = a + b + -c
    a + -c + b + -d = a + b + -c + -d
}

/// Forward differences commute with pointwise addition.
theorem fwd_diff_add[G: Mul, A: AddCommGroup](g: G, f: G -> A, k: G -> A, x: G) {
    fwd_diff(g, pointwise_add(f, k), x) = fwd_diff(g, f, x) + fwd_diff(g, k, x)
} by {
    pointwise_add(f, k, g * x) = f(g * x) + k(g * x)
    pointwise_add(f, k, x) = f(x) + k(x)
    add_sub_add_eq_sub_add_sub(f(g * x), k(g * x), f(x), k(x))
    (f(g * x) + k(g * x)) - (f(x) + k(x)) =
        (f(g * x) - f(x)) + (k(g * x) - k(x))
    fwd_diff(g, pointwise_add(f, k), x) = (f(g * x) + k(g * x)) - (f(x) + k(x))
    fwd_diff(g, f, x) = f(g * x) - f(x)
    fwd_diff(g, k, x) = k(g * x) - k(x)
}

/// Forward differences of constant functions vanish.
theorem fwd_diff_const[G: Mul, A: AddCommGroup](g: G, c: A, x: G) {
    fwd_diff(g, constant[G, A](c), x) = A.0
} by {
    constant[G, A](c, g * x) = c
    constant[G, A](c, x) = c
    fwd_diff(g, constant[G, A](c), x) = c - c
    c - c = c + -c
    c + -c = A.0
}

/// Scalar multiplication distributes over subtraction in a module.
theorem module_smul_sub[R: Ring, A: AddCommGroup](act: Module[R, A], r: R, a: A, b: A) {
    act.smul(r, a - b) = act.smul(r, a) - act.smul(r, b)
} by {
    a - b = a + -b
    module_smul_add_right(act, r, a, -b)
    act.smul(r, a + -b) = act.smul(r, a) + act.smul(r, -b)
    module_smul_neg_right(act, r, b)
    act.smul(r, -b) = -act.smul(r, b)
    act.smul(r, a + -b) = act.smul(r, a) + -act.smul(r, b)
    act.smul(r, a) - act.smul(r, b) = act.smul(r, a) + -act.smul(r, b)
}

/// Forward differences commute with pointwise scalar multiplication.
theorem fwd_diff_smul[G: Mul, R: Ring, A: AddCommGroup](
    act: Module[R, A], r: R, g: G, f: G -> A, x: G
) {
    fwd_diff(g, pointwise_smul(act, r, f), x) = pointwise_smul(act, r, fwd_diff(g, f), x)
} by {
    pointwise_smul(act, r, f, g * x) = act.smul(r, f(g * x))
    pointwise_smul(act, r, f, x) = act.smul(r, f(x))
    fwd_diff(g, pointwise_smul(act, r, f), x) = act.smul(r, f(g * x)) - act.smul(r, f(x))
    module_smul_sub(act, r, f(g * x), f(x))
    act.smul(r, f(g * x) - f(x)) = act.smul(r, f(g * x)) - act.smul(r, f(x))
    fwd_diff(g, f, x) = f(g * x) - f(x)
    pointwise_smul(act, r, fwd_diff(g, f), x) = act.smul(r, fwd_diff(g, f, x))
    pointwise_smul(act, r, fwd_diff(g, f), x) = act.smul(r, f(g * x) - f(x))
}
