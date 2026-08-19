/// Fourier series foundations: the trigonometric system, orthogonality, and
/// the Fourier coefficients.
///
/// The Fourier coefficients of a function f on [0, 2pi] are
///     a_n = (1/pi) * integral(f(x)nx.cos, 0, 2pi),
///     b_n = (1/pi) * integral(f(x)nx.sin, 0, 2pi),
/// defined below in terms of the Darboux integral API exposed by the real
/// package interface.  The main proved results of this file are the concrete
/// integral computations that underpin the orthogonality of the trigonometric
/// system on [0, 2pi]:
///   - integral((x).sincos(x)) = 0            (cross orthogonality, m = n = 1),
///   - integral(x.sin^2) = pi, integral(x.cos^2) = pi   (normalization, n = 1),
///   - integral(x.sin(2x).sin) = 0           (orthogonality, m = 1, n = 2),
///   - b_1 = (1/pi) * integral(x x.sin) = -2 for f(x) = x  (integration by parts;
///     note the sign: the integral of x x.sin over [0, 2pi] is -2pi).
/// Each value is obtained from the fundamental theorem of calculus
/// (ftc2_general) applied to an explicit antiderivative, after establishing
/// integrability through the m-Lipschitz criterion fn_integrable_gen of the
/// real package.
///
/// The general orthogonality relations for arbitrary m and n, and Parseval's
/// identity, are stated at the bottom of the file as comments: the library
/// does not yet contain the integral of nx.sin for general n (a linear
/// substitution rule for integrals), which those statements need.

from nat import Nat, from_nat, from_nat_add
from order import lte_antisymm, lte_trans, lt_trans, lt_imp_lte, lt_imp_ne
from real import Real, sin_zero, cos_zero, pi, pi_pos, sin_two_pi_zero,
    cos_two_pi_one, sin_continuous, cos_continuous, two, two_nonzero, two_positive,
    sin_add, sin_sq_add_cos_sq, sin_lipschitz, cos_lipschitz, sin_abs_le_one,
    cos_abs_le_one, integral, is_integrable, interval_contains, interval_contains_left,
    interval_contains_right, ftc2_general,
    fn_integrable_gen, is_derivative_fn, derivative_fn_identity,
    derivative_fn_constant, derivative_fn_neg, derivative_fn_add,
    derivative_fn_const_mul, derivative_fn_mul, derivative_fn_square,
    sin_is_derivative_fn, cos_derivative_is_neg_sin, abs_of_nonneg,
    continuous, continuous_pointwise_neg, continuous_pointwise_add,
    continuous_pointwise_mul, constant_function_is_continuous,
    identity_function_is_continuous, neg_lte_flip, add_comm, add_assoc,
    neg_distrib, abs_neg, neg_neg, lte_abs, abs_gte_zero, gt_zero_imp_pos,
    pos_gt_zero, sub_cancels, lte_self, neg_zero, pos_imp_eq_abs,
    from_nat_is_from_rat, mul_distrib_left, mul_distrib_right,
    mul_sub_distrib_left, mul_sub_distrib_right, mul_abs, triangle_ineq,
    mul_inverse, mul_div, mul_div_cancel, div_mul_cancel_left,
    div_mul_cancel_right, sub_zero_imp_eq, from_nat_suc_pos_real,
    zero_is_different_than_one, one_half_positive, one_half_plus_one_half,
    add_neg_eq_zero, add_zero_right, mul_nonneg
from algebra.add_ordered_group import add_le_add, add_le_add_right, add_lt_add_right
from ordered_field import mul_le_mul_of_nonneg_right, mul_le_mul_of_nonneg_left
from data.basic.function_algebra import pointwise_add, pointwise_neg, pointwise_mul,
    pointwise_mul_apply, pointwise_add_apply, pointwise_neg_apply
from data.basic.functions import identity_fn, function_extensionality

numerals Real
numerals Nat

// ---------------------------------------------------------------------------
// Two pi and the Fourier coefficients
// ---------------------------------------------------------------------------

/// Two pi, the length of the period interval.
let two_pi = pi + pi

/// The sine of n times x, for a natural number n.
define sin_nt(n: Nat, x: Real) -> Real {
    (from_nat[Real](n) * x).sin
}

/// The cosine of n times x, for a natural number n.
define cos_nt(n: Nat, x: Real) -> Real {
    (from_nat[Real](n) * x).cos
}

/// The integrand f(x)nx.cos of the n-th cosine Fourier coefficient.
define fourier_cos_integrand(f: Real -> Real, n: Nat, x: Real) -> Real {
    f(x) * cos_nt(n, x)
}

/// The integrand f(x)nx.sin of the n-th sine Fourier coefficient.
define fourier_sin_integrand(f: Real -> Real, n: Nat, x: Real) -> Real {
    f(x) * sin_nt(n, x)
}

/// The n-th cosine Fourier coefficient of f on [0, 2pi]:
/// a_n = (1/pi) * integral(f(x)nx.cos, x = 0..2pi).
define fourier_cos_coeff(f: Real -> Real, n: Nat) -> Real {
    integral(fourier_cos_integrand(f, n), Real.0, two_pi) / pi
}

/// The n-th sine Fourier coefficient of f on [0, 2pi]:
/// b_n = (1/pi) * integral(f(x)nx.sin, x = 0..2pi).
define fourier_sin_coeff(f: Real -> Real, n: Nat) -> Real {
    integral(fourier_sin_integrand(f, n), Real.0, two_pi) / pi
}

// ---------------------------------------------------------------------------
// The period interval is nonempty
// ---------------------------------------------------------------------------

/// Two pi is positive, hence nonnegative.
theorem two_pi_pos {
    Real.0 < two_pi
} by {
    pi_pos
    pi > Real.0
    add_lt_add_right(Real.0, pi, pi)
    Real.0 < pi implies Real.0 + pi < pi + pi
    Real.0 + pi < pi + pi
    Real.0 + pi = pi
    pi < pi + pi
    lt_trans(Real.0, pi, pi + pi)
    Real.0 < pi + pi
    two_pi = pi + pi
    Real.0 < two_pi
}

/// Two pi is nonnegative.
theorem two_pi_nonneg {
    Real.0 <= two_pi
} by {
    two_pi_pos
    Real.0 < two_pi
    lt_imp_lte(Real.0, two_pi)
    Real.0 <= two_pi
}

// ---------------------------------------------------------------------------
// The product functions
// ---------------------------------------------------------------------------

/// The product of sine and cosine, x ↦ (x).sincos(x).
define fn_sincos(x: Real) -> Real {
    pointwise_mul(Real.sin, Real.cos, x)
}

/// The square of sine, x ↦ x.sin^2.
define fn_sin_sq(x: Real) -> Real {
    pointwise_mul(Real.sin, Real.sin, x)
}

/// The square of cosine, x ↦ x.cos^2.
define fn_cos_sq(x: Real) -> Real {
    pointwise_mul(Real.cos, Real.cos, x)
}

/// The product x.sin(2x).sin.
define fn_sin_sin2(x: Real) -> Real {
    x.sin * (two * x).sin
}

/// The product x x.sin.
define fn_x_sin(x: Real) -> Real {
    identity_fn[Real](x) * x.sin
}

/// (2x).sin = 2 x.sin x.cos.
theorem sin_double(x: Real) {
    (two * x).sin = two * x.sin * x.cos
} by {
    two * x = x + x
    sin_add(x, x)
    (x + x).sin = x.sin * x.cos + x.cos * x.sin
    (two * x).sin = x.sin * x.cos + x.cos * x.sin
    x.cos * x.sin = x.sin * x.cos
    x.sin * x.cos + x.cos * x.sin = x.sin * x.cos + x.sin * x.cos
    Real.1 + Real.1 = two
    x.sin * x.cos + x.sin * x.cos = two * x.sin * x.cos
    (two * x).sin = two * x.sin * x.cos
}

/// Cancellation: (a - b) + b = a.
theorem sub_add_cancel(a: Real, b: Real) {
    (a - b) + b = a
} by {
    add_neg_eq_zero(b)
    b + -b = Real.0
    add_comm(b, -b)
    -b + b = Real.0
    (a - b) + b = a + -b + b
    a + -b + b = a + (-b + b)
    a + (-b + b) = a + Real.0
    add_zero_right(a)
    a + Real.0 = a
    a + (-b + b) = a
    (a - b) + b = a
}

/// Telescoping of subtractions: (a - b) + (b - c) = a - c.
theorem add_sub_sub_cancel(a: Real, b: Real, c: Real) {
    (a - b) + (b - c) = a - c
} by {
    add_assoc(a - b, b, -c)
    ((a - b) + b) + -c = (a - b) + (b + -c)
    (a - b) + (b - c) = ((a - b) + b) + -c
    sub_add_cancel(a, b)
    (a - b) + b = a
    ((a - b) + b) + -c = a + -c
    a + -c = a - c
    (a - b) + (b - c) = a - c
}

/// Cancellation of a shared summand: (a + b) - (b + c) = a - c.
theorem add_sub_add_cancel(a: Real, b: Real, c: Real) {
    (a + b) - (b + c) = a - c
} by {
    add_assoc(a, b, -b)
    (a + b) + -b = a + (b + -b)
    add_neg_eq_zero(b)
    b + -b = Real.0
    a + (b + -b) = a + Real.0
    add_zero_right(a)
    a + Real.0 = a
    (a + b) + -b = a
    add_assoc(a + b, -b, -c)
    ((a + b) + -b) + -c = (a + b) + (-b + -c)
    neg_distrib(b, c)
    -(b + c) = -b + -c
    (a + b) - (b + c) = (a + b) + (-b + -c)
    ((a + b) + -b) + -c = a + -c
    a + -c = a - c
    (a + b) - (b + c) = a - c
}


/// a + a = 2a.
theorem add_self_two(a: Real) {
    a + a = two * a
} by {
    Real.1 + Real.1 = two
    mul_distrib_left(Real.1, Real.1, a)
    (Real.1 + Real.1) * a = Real.1 * a + Real.1 * a
    Real.1 * a = a
    Real.1 * a + Real.1 * a = a + a
    (Real.1 + Real.1) * a = a + a
    two * a = a + a
    a + a = two * a
}

/// (a + b) + a - b = a + a.
theorem add_add_sub_cancel(a: Real, b: Real) {
    (a + b) + a - b = a + a
} by {
    add_comm(a + b, a)
    (a + b) + a = a + (a + b)
    add_assoc(a, a, b)
    (a + a) + b = a + (a + b)
    (a + b) + a = (a + a) + b
    sub_cancels(a + a, b)
    (a + a) + b - b = a + a
    (a + b) + a - b = (a + a) + b - b
    (a + b) + a - b = a + a
}

/// (a + b) - a + b = b + b.
theorem sub_add_add_cancel(a: Real, b: Real) {
    (a + b) - a + b = b + b
} by {
    add_assoc(a, b, -a)
    a + (b + -a) = (a + b) + -a
    add_comm(b, -a)
    b + -a = -a + b
    a + (b + -a) = a + (-a + b)
    add_assoc(a, -a, b)
    (a + -a) + b = a + (-a + b)
    add_neg_eq_zero(a)
    a + -a = Real.0
    (a + -a) + b = Real.0 + b
    Real.0 + b = b
    (a + -a) + b = b
    a + (-a + b) = b
    a + (b + -a) = b
    (a + b) + -a = b
    (a + b) - a = b
    (a + b) - a + b = b + b
}

/// x.sin^2 + x.cos^2 = 1, expanded into products.
theorem pythag_mul(x: Real) {
    x.sin * x.sin + x.cos * x.cos = Real.1
} by {
    sin_sq_add_cos_sq(x)
    x.sin.pow(Nat.2) + x.cos.pow(Nat.2) = Real.1
    x.sin.pow(Nat.2) = x.sin * x.sin
    x.cos.pow(Nat.2) = x.cos * x.cos
    x.sin * x.sin + x.cos * x.cos = Real.1
}

/// 1 + x.sin^2 - x.cos^2 = 2 x.sin^2.
theorem half_pythag_sin_sq(x: Real) {
    Real.1 + x.sin * x.sin - x.cos * x.cos = two * (x.sin * x.sin)
} by {
    pythag_mul(x)
    x.sin * x.sin + x.cos * x.cos = Real.1
    Real.1 = x.sin * x.sin + x.cos * x.cos
    Real.1 + x.sin * x.sin - x.cos * x.cos =
        (x.sin * x.sin + x.cos * x.cos) + x.sin * x.sin - x.cos * x.cos
    add_add_sub_cancel(x.sin * x.sin, x.cos * x.cos)
    (x.sin * x.sin + x.cos * x.cos) + x.sin * x.sin - x.cos * x.cos =
        x.sin * x.sin + x.sin * x.sin
    add_self_two(x.sin * x.sin)
    x.sin * x.sin + x.sin * x.sin = two * (x.sin * x.sin)
    Real.1 + x.sin * x.sin - x.cos * x.cos = two * (x.sin * x.sin)
}

/// 1 - x.sin^2 + x.cos^2 = 2 x.cos^2.
theorem half_pythag_cos_sq(x: Real) {
    Real.1 - x.sin * x.sin + x.cos * x.cos = two * (x.cos * x.cos)
} by {
    pythag_mul(x)
    x.sin * x.sin + x.cos * x.cos = Real.1
    Real.1 = x.sin * x.sin + x.cos * x.cos
    Real.1 - x.sin * x.sin + x.cos * x.cos =
        (x.sin * x.sin + x.cos * x.cos) - x.sin * x.sin + x.cos * x.cos
    sub_add_add_cancel(x.sin * x.sin, x.cos * x.cos)
    (x.sin * x.sin + x.cos * x.cos) - x.sin * x.sin + x.cos * x.cos =
        x.cos * x.cos + x.cos * x.cos
    add_self_two(x.cos * x.cos)
    x.cos * x.cos + x.cos * x.cos = two * (x.cos * x.cos)
    Real.1 - x.sin * x.sin + x.cos * x.cos = two * (x.cos * x.cos)
}

/// (1/2) * (2a) = a.
theorem half_twice(a: Real) {
    (Real.1 / two) * (two * a) = a
} by {
    two_nonzero
    two != Real.0
    mul_div_cancel(Real.1, two)
    two != Real.0 implies two * (Real.1 / two) = Real.1
    two * (Real.1 / two) = Real.1
    (Real.1 / two) * two = Real.1
    (Real.1 / two) * (two * a) = ((Real.1 / two) * two) * a
    ((Real.1 / two) * two) * a = Real.1 * a
    Real.1 * a = a
    (Real.1 / two) * (two * a) = a
}

/// (1/2) * (1 + x.sin^2 - x.cos^2) = x.sin^2.
theorem half_pythag_then_sin(x: Real) {
    (Real.1 / two) * (Real.1 + x.sin * x.sin - x.cos * x.cos) = x.sin * x.sin
} by {
    half_pythag_sin_sq(x)
    Real.1 + x.sin * x.sin - x.cos * x.cos = two * (x.sin * x.sin)
    (Real.1 / two) * (Real.1 + x.sin * x.sin - x.cos * x.cos) =
        (Real.1 / two) * (two * (x.sin * x.sin))
    half_twice(x.sin * x.sin)
    (Real.1 / two) * (two * (x.sin * x.sin)) = x.sin * x.sin
    (Real.1 / two) * (Real.1 + x.sin * x.sin - x.cos * x.cos) = x.sin * x.sin
}

/// (1/2) * (1 - x.sin^2 + x.cos^2) = x.cos^2.
theorem half_pythag_then_cos(x: Real) {
    (Real.1 / two) * (Real.1 - x.sin * x.sin + x.cos * x.cos) = x.cos * x.cos
} by {
    half_pythag_cos_sq(x)
    Real.1 - x.sin * x.sin + x.cos * x.cos = two * (x.cos * x.cos)
    (Real.1 / two) * (Real.1 - x.sin * x.sin + x.cos * x.cos) =
        (Real.1 / two) * (two * (x.cos * x.cos))
    half_twice(x.cos * x.cos)
    (Real.1 / two) * (two * (x.cos * x.cos)) = x.cos * x.cos
    (Real.1 / two) * (Real.1 - x.sin * x.sin + x.cos * x.cos) = x.cos * x.cos
}

/// 1 + 2 = from_nat(3).
theorem one_add_two_is_three {
    Real.1 + two = from_nat[Real](Nat.3)
} by {
    from_nat_add[Real](Nat.1, Nat.2)
    from_nat[Real](Nat.1 + Nat.2) = from_nat[Real](Nat.1) + from_nat[Real](Nat.2)
    Nat.1 + Nat.2 = Nat.3
    from_nat[Real](Nat.3) = from_nat[Real](Nat.1) + from_nat[Real](Nat.2)
    from_nat[Real](Nat.1) = Real.1
    from_nat[Real](Nat.2) = two
    from_nat[Real](Nat.3) = Real.1 + two
    Real.1 + two = from_nat[Real](Nat.3)
}

/// One is nonnegative.
theorem one_nonneg {
    Real.0 <= Real.1
} by {
    one_half_positive
    Real.0 < Real.one_half
    one_half_plus_one_half
    Real.one_half + Real.one_half = Real.1
    add_lt_add_right(Real.0, Real.one_half, Real.one_half)
    Real.0 < Real.one_half implies Real.0 + Real.one_half < Real.one_half + Real.one_half
    Real.0 + Real.one_half < Real.one_half + Real.one_half
    Real.0 + Real.one_half = Real.one_half
    Real.one_half < Real.one_half + Real.one_half
    Real.one_half < Real.1
    lt_trans(Real.0, Real.one_half, Real.1)
    Real.0 < Real.1
    lt_imp_lte(Real.0, Real.1)
    Real.0 <= Real.1
}

/// If a real number is at most one in absolute value, it is at most one.
theorem abs_le_one_imp_upper(a: Real) {
    a.abs <= Real.1 implies a <= Real.1
} by {
    if a.abs <= Real.1 {
        lte_abs(a)
        a <= a.abs
        lte_trans[Real](a, a.abs, Real.1)
        a <= Real.1
    }
}

/// If a real number is at most one in absolute value, it is at least negative one.
theorem abs_le_one_imp_lower(a: Real) {
    a.abs <= Real.1 implies -Real.1 <= a
} by {
    if a.abs <= Real.1 {
        lte_abs(-a)
        -a <= (-a).abs
        abs_neg(a)
        (-a).abs = a.abs
        -a <= a.abs
        lte_trans[Real](-a, a.abs, Real.1)
        -a <= Real.1
        neg_lte_flip(-a, Real.1)
        -a <= Real.1 implies -Real.1 <= -(-a)
        -Real.1 <= -(-a)
        neg_neg(a)
        -(-a) = a
        -Real.1 <= a
    }
}

/// If a real number is at most b in absolute value, it is at least -b.
theorem abs_le_imp_lower(a: Real, b: Real) {
    a.abs <= b implies -b <= a
} by {
    if a.abs <= b {
        lte_abs(-a)
        -a <= (-a).abs
        abs_neg(a)
        (-a).abs = a.abs
        -a <= a.abs
        lte_trans[Real](-a, a.abs, b)
        -a <= b
        neg_lte_flip(-a, b)
        -a <= b implies -b <= -(-a)
        -b <= -(-a)
        neg_neg(a)
        -(-a) = a
        -b <= a
    }
}

/// If a real number is at most b in absolute value, it is at most b.
theorem abs_le_imp_upper(a: Real, b: Real) {
    a.abs <= b implies a <= b
} by {
    if a.abs <= b {
        lte_abs(a)
        a <= a.abs
        lte_trans[Real](a, a.abs, b)
        a <= b
    }
}

/// If a and b are at most one in absolute value, their product is too.
theorem abs_mul_le_one(s: Real, t: Real) {
    s.abs <= Real.1 and t.abs <= Real.1 implies (s * t).abs <= Real.1
} by {
    if s.abs <= Real.1 and t.abs <= Real.1 {
        mul_abs(s, t)
        (s * t).abs = s.abs * t.abs
        abs_gte_zero(t)
        Real.0 <= t.abs
        mul_le_mul_of_nonneg_right(s.abs, Real.1, t.abs)
        s.abs * t.abs <= Real.1 * t.abs
        Real.1 * t.abs = t.abs
        s.abs * t.abs <= t.abs
        lte_trans[Real](s.abs * t.abs, t.abs, Real.1)
        (s * t).abs <= Real.1
    }
}

/// The product of sine and cosine is bounded by one in absolute value.
theorem fn_sincos_abs_le_one(x: Real) {
    fn_sincos(x).abs <= Real.1
} by {
    fn_sincos(x) = x.sin * x.cos
    sin_abs_le_one(x)
    x.sin.abs <= Real.1
    cos_abs_le_one(x)
    x.cos.abs <= Real.1
    abs_mul_le_one(x.sin, x.cos)
    (x.sin * x.cos).abs <= Real.1
    fn_sincos(x).abs <= Real.1
}

/// The square of sine is bounded by one in absolute value.
theorem fn_sin_sq_abs_le_one(x: Real) {
    fn_sin_sq(x).abs <= Real.1
} by {
    fn_sin_sq(x) = x.sin * x.sin
    sin_abs_le_one(x)
    x.sin.abs <= Real.1
    abs_mul_le_one(x.sin, x.sin)
    (x.sin * x.sin).abs <= Real.1
    fn_sin_sq(x).abs <= Real.1
}

/// The square of cosine is bounded by one in absolute value.
theorem fn_cos_sq_abs_le_one(x: Real) {
    fn_cos_sq(x).abs <= Real.1
} by {
    fn_cos_sq(x) = x.cos * x.cos
    cos_abs_le_one(x)
    x.cos.abs <= Real.1
    abs_mul_le_one(x.cos, x.cos)
    (x.cos * x.cos).abs <= Real.1
    fn_cos_sq(x).abs <= Real.1
}

/// The product x.sin(2x).sin is bounded by one in absolute value.
theorem fn_sin_sin2_abs_le_one(x: Real) {
    fn_sin_sin2(x).abs <= Real.1
} by {
    fn_sin_sin2(x) = x.sin * (two * x).sin
    sin_abs_le_one(x)
    x.sin.abs <= Real.1
    sin_abs_le_one(two * x)
    (two * x).sin.abs <= Real.1
    abs_mul_le_one(x.sin, (two * x).sin)
    (x.sin * (two * x).sin).abs <= Real.1
    fn_sin_sin2(x).abs <= Real.1
}

/// The product x x.sin is bounded by 2pi in absolute value on [0, 2pi].
theorem fn_x_sin_abs_le_two_pi(x: Real) {
    interval_contains(Real.0, two_pi, x) implies fn_x_sin(x).abs <= two_pi
} by {
    if interval_contains(Real.0, two_pi, x) {
        interval_contains_left(Real.0, two_pi, x)
        interval_contains(Real.0, two_pi, x) implies Real.0 <= x
        Real.0 <= x
        abs_of_nonneg(x)
        x >= Real.0 implies x.abs = x
        x >= Real.0
        x.abs = x
        x <= two_pi
        x.abs <= two_pi
        fn_x_sin(x) = identity_fn[Real](x) * x.sin
        identity_fn[Real](x) = x
        fn_x_sin(x) = x * x.sin
        mul_abs(x, x.sin)
        (x * x.sin).abs = x.abs * x.sin.abs
        sin_abs_le_one(x)
        x.sin.abs <= Real.1
        abs_gte_zero(x)
        Real.0 <= x.abs
        mul_le_mul_of_nonneg_right(x.abs, two_pi, x.sin.abs)
        x.abs <= two_pi implies x.abs * x.sin.abs <= two_pi * x.sin.abs
        abs_gte_zero(x.sin)
        Real.0 <= x.sin.abs
        x.abs * x.sin.abs <= two_pi * x.sin.abs
        mul_le_mul_of_nonneg_left(x.sin.abs, Real.1, two_pi)
        x.sin.abs <= Real.1 implies two_pi * x.sin.abs <= two_pi * Real.1
        two_pi * x.sin.abs <= two_pi * Real.1
        two_pi * Real.1 = two_pi
        two_pi * x.sin.abs <= two_pi
        lte_trans[Real](x.abs * x.sin.abs, two_pi * x.sin.abs, two_pi)
        x.abs * x.sin.abs <= two_pi
        (x * x.sin).abs <= two_pi
        fn_x_sin(x).abs <= two_pi
    }
}

// ---------------------------------------------------------------------------
// Lipschitz bounds of the product functions
// ---------------------------------------------------------------------------

/// |(x).sincos(x) - (y).sincos(y)| <= 2|x - y|.
theorem fn_sincos_lipschitz_m2(u: Real, v: Real) {
    (fn_sincos(u) - fn_sincos(v)).abs <= two * (u - v).abs
} by {
    fn_sincos(u) = u.sin * u.cos
    fn_sincos(v) = v.sin * v.cos
    fn_sincos(u) - fn_sincos(v) = u.sin * u.cos - v.sin * v.cos
    mul_sub_distrib_left(u.sin, v.sin, u.cos)
    (u.sin - v.sin) * u.cos = u.sin * u.cos - v.sin * u.cos
    mul_sub_distrib_left(v.sin, u.cos, v.cos)
    v.sin * (u.cos - v.cos) = v.sin * u.cos - v.sin * v.cos
    (u.sin - v.sin) * u.cos + v.sin * (u.cos - v.cos) =
        (u.sin * u.cos - v.sin * u.cos) + (v.sin * u.cos - v.sin * v.cos)
    add_sub_sub_cancel(u.sin * u.cos, v.sin * u.cos, v.sin * v.cos)
    (u.sin * u.cos - v.sin * u.cos) + (v.sin * u.cos - v.sin * v.cos) =
        u.sin * u.cos - v.sin * v.cos
    (u.sin - v.sin) * u.cos + v.sin * (u.cos - v.cos) =
        u.sin * u.cos - v.sin * v.cos
    (fn_sincos(u) - fn_sincos(v)).abs <= ((u.sin - v.sin) * u.cos + v.sin * (u.cos - v.cos)).abs
    triangle_ineq((u.sin - v.sin) * u.cos, v.sin * (u.cos - v.cos))
    ((u.sin - v.sin) * u.cos + v.sin * (u.cos - v.cos)).abs <= ((u.sin - v.sin) * u.cos).abs + (v.sin * (u.cos - v.cos)).abs
    mul_abs(u.sin - v.sin, u.cos)
    ((u.sin - v.sin) * u.cos).abs = (u.sin - v.sin).abs * u.cos.abs
    mul_abs(v.sin, u.cos - v.cos)
    (v.sin * (u.cos - v.cos)).abs = v.sin.abs * (u.cos - v.cos).abs
    (fn_sincos(u) - fn_sincos(v)).abs <= (u.sin - v.sin).abs * u.cos.abs + v.sin.abs * (u.cos - v.cos).abs
    sin_lipschitz(u, v)
    (u.sin - v.sin).abs <= (u - v).abs
    abs_gte_zero(u.cos)
    Real.0 <= u.cos.abs
    mul_le_mul_of_nonneg_right((u.sin - v.sin).abs, (u - v).abs, u.cos.abs)
    (u.sin - v.sin).abs * u.cos.abs <= (u - v).abs * u.cos.abs
    cos_abs_le_one(u)
    u.cos.abs <= Real.1
    abs_gte_zero((u - v).abs)
    Real.0 <= (u - v).abs
    mul_le_mul_of_nonneg_left(u.cos.abs, Real.1, (u - v).abs)
    (u - v).abs * u.cos.abs <= (u - v).abs * Real.1
    (u - v).abs * Real.1 = (u - v).abs
    (u - v).abs * u.cos.abs <= (u - v).abs
    lte_trans[Real]((u.sin - v.sin).abs * u.cos.abs, (u - v).abs * u.cos.abs, (u - v).abs)
    (u.sin - v.sin).abs * u.cos.abs <= (u - v).abs
    cos_lipschitz(u, v)
    (u.cos - v.cos).abs <= (u - v).abs
    abs_gte_zero(v.sin)
    Real.0 <= v.sin.abs
    mul_le_mul_of_nonneg_right((u.cos - v.cos).abs, (u - v).abs, v.sin.abs)
    (u.cos - v.cos).abs * v.sin.abs <= (u - v).abs * v.sin.abs
    sin_abs_le_one(v)
    v.sin.abs <= Real.1
    mul_le_mul_of_nonneg_left(v.sin.abs, Real.1, (u - v).abs)
    (u - v).abs * v.sin.abs <= (u - v).abs * Real.1
    (u - v).abs * v.sin.abs <= (u - v).abs
    lte_trans[Real]((u.cos - v.cos).abs * v.sin.abs, (u - v).abs * v.sin.abs, (u - v).abs)
    (u.cos - v.cos).abs * v.sin.abs <= (u - v).abs
    add_le_add(
        (u.sin - v.sin).abs * u.cos.abs, (u - v).abs,
        (u.cos - v.cos).abs * v.sin.abs, (u - v).abs)
    (u.sin - v.sin).abs * u.cos.abs + (u.cos - v.cos).abs * v.sin.abs <= (u - v).abs + (u - v).abs
    (u - v).abs + (u - v).abs = two * (u - v).abs
    (u.sin - v.sin).abs * u.cos.abs + (u.cos - v.cos).abs * v.sin.abs <= two * (u - v).abs
    lte_trans[Real]((fn_sincos(u) - fn_sincos(v)).abs,
        (u.sin - v.sin).abs * u.cos.abs + (u.cos - v.cos).abs * v.sin.abs,
        two * (u - v).abs)
    (fn_sincos(u) - fn_sincos(v)).abs <= two * (u - v).abs
}

/// |u.sin + v.sin| <= 2.
theorem sin_add_abs_le_two(u: Real, v: Real) {
    (u.sin + v.sin).abs <= two
} by {
    triangle_ineq(u.sin, v.sin)
    (u.sin + v.sin).abs <= u.sin.abs + v.sin.abs
    sin_abs_le_one(u)
    u.sin.abs <= Real.1
    sin_abs_le_one(v)
    v.sin.abs <= Real.1
    add_le_add(u.sin.abs, Real.1, v.sin.abs, Real.1)
    u.sin.abs + v.sin.abs <= Real.1 + Real.1
    Real.1 + Real.1 = two
    u.sin.abs + v.sin.abs <= two
    lte_trans[Real]((u.sin + v.sin).abs, u.sin.abs + v.sin.abs, two)
    (u.sin + v.sin).abs <= two
}

/// |x.sin^2 - y.sin^2| <= 2|x - y|.
theorem fn_sin_sq_lipschitz_m2(u: Real, v: Real) {
    (fn_sin_sq(u) - fn_sin_sq(v)).abs <= two * (u - v).abs
} by {
    fn_sin_sq(u) = u.sin * u.sin
    fn_sin_sq(v) = v.sin * v.sin
    fn_sin_sq(u) - fn_sin_sq(v) = u.sin * u.sin - v.sin * v.sin
    mul_sub_distrib_left(u.sin, v.sin, u.sin + v.sin)
    (u.sin - v.sin) * (u.sin + v.sin) = u.sin * (u.sin + v.sin) - v.sin * (u.sin + v.sin)
    mul_distrib_right(u.sin, u.sin, v.sin)
    u.sin * (u.sin + v.sin) = u.sin * u.sin + u.sin * v.sin
    mul_distrib_right(v.sin, u.sin, v.sin)
    v.sin * (u.sin + v.sin) = v.sin * u.sin + v.sin * v.sin
    (u.sin - v.sin) * (u.sin + v.sin) =
        (u.sin * u.sin + u.sin * v.sin) - (v.sin * u.sin + v.sin * v.sin)
    u.sin * v.sin = v.sin * u.sin
    (u.sin * u.sin + u.sin * v.sin) - (v.sin * u.sin + v.sin * v.sin) =
        (u.sin * u.sin + v.sin * u.sin) - (v.sin * u.sin + v.sin * v.sin)
    add_sub_add_cancel(u.sin * u.sin, v.sin * u.sin, v.sin * v.sin)
    (u.sin * u.sin + v.sin * u.sin) - (v.sin * u.sin + v.sin * v.sin) =
        u.sin * u.sin - v.sin * v.sin
    (u.sin - v.sin) * (u.sin + v.sin) = u.sin * u.sin - v.sin * v.sin
    (fn_sin_sq(u) - fn_sin_sq(v)).abs <= ((u.sin - v.sin) * (u.sin + v.sin)).abs
    mul_abs(u.sin - v.sin, u.sin + v.sin)
    ((u.sin - v.sin) * (u.sin + v.sin)).abs =
        (u.sin - v.sin).abs * (u.sin + v.sin).abs
    (fn_sin_sq(u) - fn_sin_sq(v)).abs <= (u.sin - v.sin).abs * (u.sin + v.sin).abs
    sin_lipschitz(u, v)
    (u.sin - v.sin).abs <= (u - v).abs
    abs_gte_zero(u.sin + v.sin)
    Real.0 <= (u.sin + v.sin).abs
    mul_le_mul_of_nonneg_right((u.sin - v.sin).abs, (u - v).abs, (u.sin + v.sin).abs)
    (u.sin - v.sin).abs * (u.sin + v.sin).abs <= (u - v).abs * (u.sin + v.sin).abs
    sin_add_abs_le_two(u, v)
    (u.sin + v.sin).abs <= two
    abs_gte_zero((u - v).abs)
    Real.0 <= (u - v).abs
    mul_le_mul_of_nonneg_left((u.sin + v.sin).abs, two, (u - v).abs)
    (u - v).abs * (u.sin + v.sin).abs <= (u - v).abs * two
    (u - v).abs * two = two * (u - v).abs
    (u - v).abs * (u.sin + v.sin).abs <= two * (u - v).abs
    lte_trans[Real]((u.sin - v.sin).abs * (u.sin + v.sin).abs,
        (u - v).abs * (u.sin + v.sin).abs, two * (u - v).abs)
    (u.sin - v.sin).abs * (u.sin + v.sin).abs <= two * (u - v).abs
    lte_trans[Real]((fn_sin_sq(u) - fn_sin_sq(v)).abs,
        (u.sin - v.sin).abs * (u.sin + v.sin).abs,
        two * (u - v).abs)
    (fn_sin_sq(u) - fn_sin_sq(v)).abs <= two * (u - v).abs
}

/// |u.cos + v.cos| <= 2.
theorem cos_add_abs_le_two(u: Real, v: Real) {
    (u.cos + v.cos).abs <= two
} by {
    triangle_ineq(u.cos, v.cos)
    (u.cos + v.cos).abs <= u.cos.abs + v.cos.abs
    cos_abs_le_one(u)
    u.cos.abs <= Real.1
    cos_abs_le_one(v)
    v.cos.abs <= Real.1
    add_le_add(u.cos.abs, Real.1, v.cos.abs, Real.1)
    u.cos.abs + v.cos.abs <= Real.1 + Real.1
    Real.1 + Real.1 = two
    u.cos.abs + v.cos.abs <= two
    lte_trans[Real]((u.cos + v.cos).abs, u.cos.abs + v.cos.abs, two)
    (u.cos + v.cos).abs <= two
}

/// |x.cos^2 - y.cos^2| <= 2|x - y|.
theorem fn_cos_sq_lipschitz_m2(u: Real, v: Real) {
    (fn_cos_sq(u) - fn_cos_sq(v)).abs <= two * (u - v).abs
} by {
    fn_cos_sq(u) = u.cos * u.cos
    fn_cos_sq(v) = v.cos * v.cos
    fn_cos_sq(u) - fn_cos_sq(v) = u.cos * u.cos - v.cos * v.cos
    mul_sub_distrib_left(u.cos, v.cos, u.cos + v.cos)
    (u.cos - v.cos) * (u.cos + v.cos) = u.cos * (u.cos + v.cos) - v.cos * (u.cos + v.cos)
    mul_distrib_right(u.cos, u.cos, v.cos)
    u.cos * (u.cos + v.cos) = u.cos * u.cos + u.cos * v.cos
    mul_distrib_right(v.cos, u.cos, v.cos)
    v.cos * (u.cos + v.cos) = v.cos * u.cos + v.cos * v.cos
    (u.cos - v.cos) * (u.cos + v.cos) =
        (u.cos * u.cos + u.cos * v.cos) - (v.cos * u.cos + v.cos * v.cos)
    u.cos * v.cos = v.cos * u.cos
    (u.cos * u.cos + u.cos * v.cos) - (v.cos * u.cos + v.cos * v.cos) =
        (u.cos * u.cos + v.cos * u.cos) - (v.cos * u.cos + v.cos * v.cos)
    add_sub_add_cancel(u.cos * u.cos, v.cos * u.cos, v.cos * v.cos)
    (u.cos * u.cos + v.cos * u.cos) - (v.cos * u.cos + v.cos * v.cos) =
        u.cos * u.cos - v.cos * v.cos
    (u.cos - v.cos) * (u.cos + v.cos) = u.cos * u.cos - v.cos * v.cos
    (fn_cos_sq(u) - fn_cos_sq(v)).abs <= ((u.cos - v.cos) * (u.cos + v.cos)).abs
    mul_abs(u.cos - v.cos, u.cos + v.cos)
    ((u.cos - v.cos) * (u.cos + v.cos)).abs =
        (u.cos - v.cos).abs * (u.cos + v.cos).abs
    (fn_cos_sq(u) - fn_cos_sq(v)).abs <= (u.cos - v.cos).abs * (u.cos + v.cos).abs
    cos_lipschitz(u, v)
    (u.cos - v.cos).abs <= (u - v).abs
    abs_gte_zero(u.cos + v.cos)
    Real.0 <= (u.cos + v.cos).abs
    mul_le_mul_of_nonneg_right((u.cos - v.cos).abs, (u - v).abs, (u.cos + v.cos).abs)
    (u.cos - v.cos).abs * (u.cos + v.cos).abs <= (u - v).abs * (u.cos + v.cos).abs
    cos_add_abs_le_two(u, v)
    (u.cos + v.cos).abs <= two
    abs_gte_zero((u - v).abs)
    Real.0 <= (u - v).abs
    mul_le_mul_of_nonneg_left((u.cos + v.cos).abs, two, (u - v).abs)
    (u - v).abs * (u.cos + v.cos).abs <= (u - v).abs * two
    (u - v).abs * two = two * (u - v).abs
    (u - v).abs * (u.cos + v.cos).abs <= two * (u - v).abs
    lte_trans[Real]((u.cos - v.cos).abs * (u.cos + v.cos).abs,
        (u - v).abs * (u.cos + v.cos).abs, two * (u - v).abs)
    (u.cos - v.cos).abs * (u.cos + v.cos).abs <= two * (u - v).abs
    lte_trans[Real]((fn_cos_sq(u) - fn_cos_sq(v)).abs,
        (u.cos - v.cos).abs * (u.cos + v.cos).abs,
        two * (u - v).abs)
    (fn_cos_sq(u) - fn_cos_sq(v)).abs <= two * (u - v).abs
}

/// |x.sin(2x).sin - y.sin(2y).sin| <= 3|x - y|.
theorem fn_sin_sin2_lipschitz_m3(u: Real, v: Real) {
    (fn_sin_sin2(u) - fn_sin_sin2(v)).abs <= from_nat[Real](Nat.3) * (u - v).abs
} by {
    fn_sin_sin2(u) = u.sin * (two * u).sin
    fn_sin_sin2(v) = v.sin * (two * v).sin
    fn_sin_sin2(u) - fn_sin_sin2(v) =
        u.sin * (two * u).sin - v.sin * (two * v).sin
    mul_sub_distrib_left(u.sin, v.sin, (two * u).sin)
    (u.sin - v.sin) * (two * u).sin = u.sin * (two * u).sin - v.sin * (two * u).sin
    mul_sub_distrib_left(v.sin, (two * u).sin, (two * v).sin)
    v.sin * ((two * u).sin - (two * v).sin) = v.sin * (two * u).sin - v.sin * (two * v).sin
    (u.sin - v.sin) * (two * u).sin + v.sin * ((two * u).sin - (two * v).sin) =
        (u.sin * (two * u).sin - v.sin * (two * u).sin) + (v.sin * (two * u).sin - v.sin * (two * v).sin)
    add_sub_sub_cancel(u.sin * (two * u).sin, v.sin * (two * u).sin, v.sin * (two * v).sin)
    (u.sin * (two * u).sin - v.sin * (two * u).sin) + (v.sin * (two * u).sin - v.sin * (two * v).sin) =
        u.sin * (two * u).sin - v.sin * (two * v).sin
    (u.sin - v.sin) * (two * u).sin + v.sin * ((two * u).sin - (two * v).sin) =
        u.sin * (two * u).sin - v.sin * (two * v).sin
    (fn_sin_sin2(u) - fn_sin_sin2(v)).abs <= ((u.sin - v.sin) * (two * u).sin + v.sin * ((two * u).sin - (two * v).sin)).abs
    triangle_ineq((u.sin - v.sin) * (two * u).sin, v.sin * ((two * u).sin - (two * v).sin))
    ((u.sin - v.sin) * (two * u).sin + v.sin * ((two * u).sin - (two * v).sin)).abs <= ((u.sin - v.sin) * (two * u).sin).abs +
            (v.sin * ((two * u).sin - (two * v).sin)).abs
    mul_abs(u.sin - v.sin, (two * u).sin)
    ((u.sin - v.sin) * (two * u).sin).abs = (u.sin - v.sin).abs * (two * u).sin.abs
    mul_abs(v.sin, (two * u).sin - (two * v).sin)
    (v.sin * ((two * u).sin - (two * v).sin)).abs =
        v.sin.abs * ((two * u).sin - (two * v).sin).abs
    (fn_sin_sin2(u) - fn_sin_sin2(v)).abs <= (u.sin - v.sin).abs * (two * u).sin.abs +
            v.sin.abs * ((two * u).sin - (two * v).sin).abs
    sin_lipschitz(u, v)
    (u.sin - v.sin).abs <= (u - v).abs
    abs_gte_zero((two * u).sin)
    Real.0 <= (two * u).sin.abs
    mul_le_mul_of_nonneg_right((u.sin - v.sin).abs, (u - v).abs, (two * u).sin.abs)
    (u.sin - v.sin).abs * (two * u).sin.abs <= (u - v).abs * (two * u).sin.abs
    sin_abs_le_one(two * u)
    (two * u).sin.abs <= Real.1
    abs_gte_zero((u - v).abs)
    Real.0 <= (u - v).abs
    mul_le_mul_of_nonneg_left((two * u).sin.abs, Real.1, (u - v).abs)
    (u - v).abs * (two * u).sin.abs <= (u - v).abs * Real.1
    (u - v).abs * Real.1 = (u - v).abs
    (u - v).abs * (two * u).sin.abs <= (u - v).abs
    lte_trans[Real]((u.sin - v.sin).abs * (two * u).sin.abs,
        (u - v).abs * (two * u).sin.abs, (u - v).abs)
    (u.sin - v.sin).abs * (two * u).sin.abs <= (u - v).abs
    sin_lipschitz(two * u, two * v)
    ((two * u).sin - (two * v).sin).abs <= (two * u - two * v).abs
    two * u - two * v = two * (u - v)
    ((two * u).sin - (two * v).sin).abs <= (two * (u - v)).abs
    mul_abs(two, u - v)
    (two * (u - v)).abs = two.abs * (u - v).abs
    two_positive
    two.is_positive
    pos_gt_zero(two)
    two > Real.0
    gt_zero_imp_pos(two)
    pos_imp_eq_abs(two)
    two.is_positive implies two.abs = two
    two.abs = two
    (two * (u - v)).abs = two * (u - v).abs
    ((two * u).sin - (two * v).sin).abs <= two * (u - v).abs
    abs_gte_zero(v.sin)
    Real.0 <= v.sin.abs
    mul_le_mul_of_nonneg_right(((two * u).sin - (two * v).sin).abs,
        two * (u - v).abs, v.sin.abs)
    ((two * u).sin - (two * v).sin).abs * v.sin.abs <= (two * (u - v).abs) * v.sin.abs
    sin_abs_le_one(v)
    v.sin.abs <= Real.1
    abs_gte_zero((u - v).abs)
    Real.0 <= (u - v).abs
    two_positive
    two.is_positive
    pos_gt_zero(two)
    two > Real.0
    lt_imp_lte(Real.0, two)
    Real.0 <= two
    mul_nonneg(two, (u - v).abs)
    two >= Real.0 and (u - v).abs >= Real.0 implies two * (u - v).abs >= Real.0
    two * (u - v).abs >= Real.0
    Real.0 <= two * (u - v).abs
    mul_le_mul_of_nonneg_left(v.sin.abs, Real.1, two * (u - v).abs)
    (two * (u - v).abs) * v.sin.abs <= (two * (u - v).abs) * Real.1
    (two * (u - v).abs) * Real.1 = two * (u - v).abs
    (two * (u - v).abs) * v.sin.abs <= two * (u - v).abs
    lte_trans[Real](((two * u).sin - (two * v).sin).abs * v.sin.abs,
        (two * (u - v).abs) * v.sin.abs, two * (u - v).abs)
    ((two * u).sin - (two * v).sin).abs * v.sin.abs <= two * (u - v).abs
    add_le_add(
        (u.sin - v.sin).abs * (two * u).sin.abs, (u - v).abs,
        ((two * u).sin - (two * v).sin).abs * v.sin.abs, two * (u - v).abs)
    (u.sin - v.sin).abs * (two * u).sin.abs +
        ((two * u).sin - (two * v).sin).abs * v.sin.abs <= (u - v).abs + two * (u - v).abs
    one_add_two_is_three
    Real.1 + two = from_nat[Real](Nat.3)
    mul_distrib_left(Real.1, two, (u - v).abs)
    (Real.1 + two) * (u - v).abs = Real.1 * (u - v).abs + two * (u - v).abs
    Real.1 * (u - v).abs = (u - v).abs
    from_nat[Real](Nat.3) * (u - v).abs = (u - v).abs + two * (u - v).abs
    (u - v).abs + two * (u - v).abs = from_nat[Real](Nat.3) * (u - v).abs
    (u.sin - v.sin).abs * (two * u).sin.abs +
        ((two * u).sin - (two * v).sin).abs * v.sin.abs <= from_nat[Real](Nat.3) * (u - v).abs
    lte_trans[Real]((fn_sin_sin2(u) - fn_sin_sin2(v)).abs,
        (u.sin - v.sin).abs * (two * u).sin.abs +
            ((two * u).sin - (two * v).sin).abs * v.sin.abs,
        from_nat[Real](Nat.3) * (u - v).abs)
    (fn_sin_sin2(u) - fn_sin_sin2(v)).abs <= from_nat[Real](Nat.3) * (u - v).abs
}

/// |x x.sin - y y.sin| <= (1 + 2pi)|x - y| on [0, 2pi].
theorem fn_x_sin_lipschitz_on(u: Real, v: Real) {
    interval_contains(Real.0, two_pi, u) and interval_contains(Real.0, two_pi, v)
    implies
    (fn_x_sin(u) - fn_x_sin(v)).abs <= (Real.1 + two_pi) * (u - v).abs
} by {
    if interval_contains(Real.0, two_pi, u) and interval_contains(Real.0, two_pi, v) {
        fn_x_sin(u) = identity_fn[Real](u) * u.sin
        identity_fn[Real](u) = u
        fn_x_sin(u) = u * u.sin
        fn_x_sin(v) = identity_fn[Real](v) * v.sin
        identity_fn[Real](v) = v
        fn_x_sin(v) = v * v.sin
        fn_x_sin(u) - fn_x_sin(v) = u * u.sin - v * v.sin
        mul_sub_distrib_left(u, v, u.sin)
        (u - v) * u.sin = u * u.sin - v * u.sin
        mul_sub_distrib_left(v, u.sin, v.sin)
        v * (u.sin - v.sin) = v * u.sin - v * v.sin
        (u - v) * u.sin + v * (u.sin - v.sin) =
            (u * u.sin - v * u.sin) + (v * u.sin - v * v.sin)
        add_sub_sub_cancel(u * u.sin, v * u.sin, v * v.sin)
        (u * u.sin - v * u.sin) + (v * u.sin - v * v.sin) =
            u * u.sin - v * v.sin
        (u - v) * u.sin + v * (u.sin - v.sin) =
            u * u.sin - v * v.sin
        (fn_x_sin(u) - fn_x_sin(v)).abs <= ((u - v) * u.sin + v * (u.sin - v.sin)).abs
        triangle_ineq((u - v) * u.sin, v * (u.sin - v.sin))
        ((u - v) * u.sin + v * (u.sin - v.sin)).abs <= ((u - v) * u.sin).abs + (v * (u.sin - v.sin)).abs
        mul_abs(u - v, u.sin)
        ((u - v) * u.sin).abs = (u - v).abs * u.sin.abs
        mul_abs(v, u.sin - v.sin)
        (v * (u.sin - v.sin)).abs = v.abs * (u.sin - v.sin).abs
        (fn_x_sin(u) - fn_x_sin(v)).abs <= (u - v).abs * u.sin.abs + v.abs * (u.sin - v.sin).abs
        sin_abs_le_one(u)
        u.sin.abs <= Real.1
        abs_gte_zero((u - v).abs)
        Real.0 <= (u - v).abs
        mul_le_mul_of_nonneg_left(u.sin.abs, Real.1, (u - v).abs)
        (u - v).abs * u.sin.abs <= (u - v).abs * Real.1
        (u - v).abs * Real.1 = (u - v).abs
        (u - v).abs * u.sin.abs <= (u - v).abs
        interval_contains_left(Real.0, two_pi, v)
        interval_contains(Real.0, two_pi, v) implies Real.0 <= v
        Real.0 <= v
        interval_contains_right(Real.0, two_pi, v)
        interval_contains(Real.0, two_pi, v) implies v <= two_pi
        v <= two_pi
        abs_of_nonneg(v)
        v >= Real.0 implies v.abs = v
        v >= Real.0
        v.abs = v
        v.abs <= two_pi
        sin_lipschitz(u, v)
        (u.sin - v.sin).abs <= (u - v).abs
        abs_gte_zero(v.abs)
        Real.0 <= v.abs
        mul_le_mul_of_nonneg_right((u.sin - v.sin).abs, (u - v).abs, v.abs)
        (u.sin - v.sin).abs * v.abs <= (u - v).abs * v.abs
        abs_gte_zero((u - v).abs)
        Real.0 <= (u - v).abs
        mul_le_mul_of_nonneg_left(v.abs, two_pi, (u - v).abs)
        v.abs <= two_pi implies (u - v).abs * v.abs <= (u - v).abs * two_pi
        (u - v).abs * v.abs <= (u - v).abs * two_pi
        (u - v).abs * two_pi = two_pi * (u - v).abs
        (u - v).abs * v.abs <= two_pi * (u - v).abs
        lte_trans[Real]((u.sin - v.sin).abs * v.abs, (u - v).abs * v.abs,
            two_pi * (u - v).abs)
        (u.sin - v.sin).abs * v.abs <= two_pi * (u - v).abs
        add_le_add((u - v).abs * u.sin.abs, (u - v).abs,
            (u.sin - v.sin).abs * v.abs, two_pi * (u - v).abs)
        (u - v).abs * u.sin.abs + (u.sin - v.sin).abs * v.abs <= (u - v).abs + two_pi * (u - v).abs
        (u - v).abs + two_pi * (u - v).abs = (Real.1 + two_pi) * (u - v).abs
        (u - v).abs * u.sin.abs + (u.sin - v.sin).abs * v.abs <= (Real.1 + two_pi) * (u - v).abs
        lte_trans[Real]((fn_x_sin(u) - fn_x_sin(v)).abs,
            (u - v).abs * u.sin.abs + (u.sin - v.sin).abs * v.abs,
            (Real.1 + two_pi) * (u - v).abs)
        (fn_x_sin(u) - fn_x_sin(v)).abs <= (Real.1 + two_pi) * (u - v).abs
    }
}

// ---------------------------------------------------------------------------
// Antiderivatives of the product functions
// ---------------------------------------------------------------------------

/// An antiderivative of (x).sincos(x): x ↦ x.sin^2 / 2.
define g_sincos(x: Real) -> Real {
    pointwise_mul(constant[Real, Real](Real.1 / two), pointwise_mul(Real.sin, Real.sin), x)
}

/// An antiderivative of x.sin^2: x ↦ (x - (x).sincos(x)) / 2.
define g_sin_sq(x: Real) -> Real {
    pointwise_mul(constant[Real, Real](Real.1 / two),
        pointwise_add(identity_fn[Real], pointwise_neg(pointwise_mul(Real.sin, Real.cos))), x)
}

/// An antiderivative of x.cos^2: x ↦ (x + (x).sincos(x)) / 2.
define g_cos_sq(x: Real) -> Real {
    pointwise_mul(constant[Real, Real](Real.1 / two),
        pointwise_add(identity_fn[Real], pointwise_mul(Real.sin, Real.cos)), x)
}

/// An antiderivative of x.sin(2x).sin: x ↦ 2 x.sin^3 / 3.
define g_sin_sin2(x: Real) -> Real {
    pointwise_mul(constant[Real, Real](two / from_nat[Real](Nat.3)),
        pointwise_mul(pointwise_mul(Real.sin, Real.sin), Real.sin), x)
}

/// An antiderivative of x x.sin: x ↦ x.sin - x x.cos.
define g_x_sin(x: Real) -> Real {
    pointwise_add(Real.sin, pointwise_neg(pointwise_mul(identity_fn[Real], Real.cos)), x)
}

// ---------------------------------------------------------------------------
// Derivative of the antiderivatives
// ---------------------------------------------------------------------------

/// The derivative of g_sincos is fn_sincos.
theorem g_sincos_has_derivative {
    is_derivative_fn(g_sincos, fn_sincos)
} by {
    sin_is_derivative_fn
    is_derivative_fn(Real.sin, Real.cos)
    derivative_fn_square(Real.sin, Real.cos)
    is_derivative_fn(pointwise_mul(Real.sin, Real.sin),
        pointwise_add(pointwise_mul(Real.sin, Real.cos), pointwise_mul(Real.sin, Real.cos)))
    derivative_fn_const_mul(Real.1 / two, pointwise_mul(Real.sin, Real.sin),
        pointwise_add(pointwise_mul(Real.sin, Real.cos), pointwise_mul(Real.sin, Real.cos)))
    is_derivative_fn(
        pointwise_mul(constant[Real, Real](Real.1 / two), pointwise_mul(Real.sin, Real.sin)),
        pointwise_mul(constant[Real, Real](Real.1 / two),
            pointwise_add(pointwise_mul(Real.sin, Real.cos), pointwise_mul(Real.sin, Real.cos))))
    forall(x: Real) {
        pointwise_mul_apply(constant[Real, Real](Real.1 / two),
            pointwise_add(pointwise_mul(Real.sin, Real.cos), pointwise_mul(Real.sin, Real.cos)), x)
        pointwise_mul(constant[Real, Real](Real.1 / two),
            pointwise_add(pointwise_mul(Real.sin, Real.cos), pointwise_mul(Real.sin, Real.cos)), x) =
            constant[Real, Real](Real.1 / two, x) *
                pointwise_add(pointwise_mul(Real.sin, Real.cos), pointwise_mul(Real.sin, Real.cos), x)
        constant[Real, Real](Real.1 / two, x) = Real.1 / two
        pointwise_add_apply(pointwise_mul(Real.sin, Real.cos), pointwise_mul(Real.sin, Real.cos), x)
        pointwise_add(pointwise_mul(Real.sin, Real.cos), pointwise_mul(Real.sin, Real.cos), x) =
            pointwise_mul(Real.sin, Real.cos, x) + pointwise_mul(Real.sin, Real.cos, x)
        pointwise_mul_apply(Real.sin, Real.cos, x)
        pointwise_mul(Real.sin, Real.cos, x) = x.sin * x.cos
        pointwise_mul(constant[Real, Real](Real.1 / two),
            pointwise_add(pointwise_mul(Real.sin, Real.cos), pointwise_mul(Real.sin, Real.cos)), x) =
            (Real.1 / two) * (x.sin * x.cos + x.sin * x.cos)
        x.sin * x.cos + x.sin * x.cos = two * (x.sin * x.cos)
        two_nonzero
        two != Real.0
        mul_div_cancel(Real.1, two)
        two != Real.0 implies two * (Real.1 / two) = Real.1
        two * (Real.1 / two) = Real.1
        (Real.1 / two) * two = Real.1
        (Real.1 / two) * (two * (x.sin * x.cos)) = x.sin * x.cos
        (Real.1 / two) * (x.sin * x.cos + x.sin * x.cos) = x.sin * x.cos
        pointwise_mul(constant[Real, Real](Real.1 / two),
            pointwise_add(pointwise_mul(Real.sin, Real.cos), pointwise_mul(Real.sin, Real.cos)), x) =
            x.sin * x.cos
        fn_sincos(x) = x.sin * x.cos
        pointwise_mul(constant[Real, Real](Real.1 / two),
            pointwise_add(pointwise_mul(Real.sin, Real.cos), pointwise_mul(Real.sin, Real.cos)), x) =
            fn_sincos(x)
    }
    function_extensionality(pointwise_mul(constant[Real, Real](Real.1 / two),
        pointwise_add(pointwise_mul(Real.sin, Real.cos), pointwise_mul(Real.sin, Real.cos))), fn_sincos)
    pointwise_mul(constant[Real, Real](Real.1 / two),
        pointwise_add(pointwise_mul(Real.sin, Real.cos), pointwise_mul(Real.sin, Real.cos))) = fn_sincos
    is_derivative_fn(pointwise_mul(constant[Real, Real](Real.1 / two), pointwise_mul(Real.sin, Real.sin)),
        fn_sincos)
    is_derivative_fn(g_sincos, fn_sincos)
}

/// The derivative of g_sin_sq is fn_sin_sq.
theorem g_sin_sq_has_derivative {
    is_derivative_fn(g_sin_sq, fn_sin_sq)
} by {
    sin_is_derivative_fn
    is_derivative_fn(Real.sin, Real.cos)
    cos_derivative_is_neg_sin
    is_derivative_fn(Real.cos, pointwise_neg(Real.sin))
    derivative_fn_mul(Real.sin, Real.cos, Real.cos, pointwise_neg(Real.sin))
    is_derivative_fn(pointwise_mul(Real.sin, Real.cos),
        pointwise_add(pointwise_mul(Real.sin, pointwise_neg(Real.sin)), pointwise_mul(Real.cos, Real.cos)))
    derivative_fn_neg(pointwise_mul(Real.sin, Real.cos),
        pointwise_add(pointwise_mul(Real.sin, pointwise_neg(Real.sin)), pointwise_mul(Real.cos, Real.cos)))
    is_derivative_fn(pointwise_neg(pointwise_mul(Real.sin, Real.cos)),
        pointwise_neg(pointwise_add(pointwise_mul(Real.sin, pointwise_neg(Real.sin)), pointwise_mul(Real.cos, Real.cos))))
    derivative_fn_identity
    is_derivative_fn(identity_fn[Real], constant[Real, Real](Real.1))
    derivative_fn_add(identity_fn[Real], pointwise_neg(pointwise_mul(Real.sin, Real.cos)),
        constant[Real, Real](Real.1),
        pointwise_neg(pointwise_add(pointwise_mul(Real.sin, pointwise_neg(Real.sin)), pointwise_mul(Real.cos, Real.cos))))
    is_derivative_fn(pointwise_add(identity_fn[Real], pointwise_neg(pointwise_mul(Real.sin, Real.cos))),
        pointwise_add(constant[Real, Real](Real.1),
            pointwise_neg(pointwise_add(pointwise_mul(Real.sin, pointwise_neg(Real.sin)), pointwise_mul(Real.cos, Real.cos)))))
    derivative_fn_const_mul(Real.1 / two,
        pointwise_add(identity_fn[Real], pointwise_neg(pointwise_mul(Real.sin, Real.cos))),
        pointwise_add(constant[Real, Real](Real.1),
            pointwise_neg(pointwise_add(pointwise_mul(Real.sin, pointwise_neg(Real.sin)), pointwise_mul(Real.cos, Real.cos)))))
    is_derivative_fn(g_sin_sq,
        pointwise_mul(constant[Real, Real](Real.1 / two),
            pointwise_add(constant[Real, Real](Real.1),
                pointwise_neg(pointwise_add(pointwise_mul(Real.sin, pointwise_neg(Real.sin)), pointwise_mul(Real.cos, Real.cos))))))
    forall(x: Real) {
        pointwise_mul_apply(constant[Real, Real](Real.1 / two),
            pointwise_add(constant[Real, Real](Real.1),
                pointwise_neg(pointwise_add(pointwise_mul(Real.sin, pointwise_neg(Real.sin)), pointwise_mul(Real.cos, Real.cos)))), x)
        pointwise_mul(constant[Real, Real](Real.1 / two),
            pointwise_add(constant[Real, Real](Real.1),
                pointwise_neg(pointwise_add(pointwise_mul(Real.sin, pointwise_neg(Real.sin)), pointwise_mul(Real.cos, Real.cos)))), x) =
            constant[Real, Real](Real.1 / two, x) * pointwise_add(constant[Real, Real](Real.1),
                pointwise_neg(pointwise_add(pointwise_mul(Real.sin, pointwise_neg(Real.sin)), pointwise_mul(Real.cos, Real.cos))), x)
        constant[Real, Real](Real.1 / two, x) = Real.1 / two
        pointwise_add_apply(constant[Real, Real](Real.1),
            pointwise_neg(pointwise_add(pointwise_mul(Real.sin, pointwise_neg(Real.sin)), pointwise_mul(Real.cos, Real.cos))), x)
        pointwise_add(constant[Real, Real](Real.1),
            pointwise_neg(pointwise_add(pointwise_mul(Real.sin, pointwise_neg(Real.sin)), pointwise_mul(Real.cos, Real.cos))), x) =
            constant[Real, Real](Real.1, x) + pointwise_neg(pointwise_add(pointwise_mul(Real.sin, pointwise_neg(Real.sin)), pointwise_mul(Real.cos, Real.cos)), x)
        constant[Real, Real](Real.1, x) = Real.1
        pointwise_neg_apply(pointwise_add(pointwise_mul(Real.sin, pointwise_neg(Real.sin)), pointwise_mul(Real.cos, Real.cos)), x)
        pointwise_neg(pointwise_add(pointwise_mul(Real.sin, pointwise_neg(Real.sin)), pointwise_mul(Real.cos, Real.cos)), x) =
            -(pointwise_add(pointwise_mul(Real.sin, pointwise_neg(Real.sin)), pointwise_mul(Real.cos, Real.cos), x))
        pointwise_add_apply(pointwise_mul(Real.sin, pointwise_neg(Real.sin)), pointwise_mul(Real.cos, Real.cos), x)
        pointwise_add(pointwise_mul(Real.sin, pointwise_neg(Real.sin)), pointwise_mul(Real.cos, Real.cos), x) =
            pointwise_mul(Real.sin, pointwise_neg(Real.sin), x) + pointwise_mul(Real.cos, Real.cos, x)
        pointwise_mul_apply(Real.sin, pointwise_neg(Real.sin), x)
        pointwise_mul(Real.sin, pointwise_neg(Real.sin), x) = x.sin * pointwise_neg(Real.sin, x)
        pointwise_neg_apply(Real.sin, x)
        pointwise_neg(Real.sin, x) = -x.sin
        pointwise_mul_apply(Real.cos, Real.cos, x)
        pointwise_mul(Real.cos, Real.cos, x) = x.cos * x.cos
        pointwise_mul(constant[Real, Real](Real.1 / two),
            pointwise_add(constant[Real, Real](Real.1),
                pointwise_neg(pointwise_add(pointwise_mul(Real.sin, pointwise_neg(Real.sin)), pointwise_mul(Real.cos, Real.cos)))), x) =
            (Real.1 / two) * (Real.1 + -(x.sin * pointwise_neg(Real.sin, x) + x.cos * x.cos))
        pointwise_neg(Real.sin, x) = -x.sin
        (Real.1 / two) * (Real.1 + -(x.sin * pointwise_neg(Real.sin, x) + x.cos * x.cos)) =
            (Real.1 / two) * (Real.1 + -(x.sin * -x.sin + x.cos * x.cos))
        x.sin * -x.sin = -(x.sin * x.sin)
        (Real.1 / two) * (Real.1 + -(x.sin * -x.sin + x.cos * x.cos)) =
            (Real.1 / two) * (Real.1 + x.sin * x.sin - x.cos * x.cos)
        half_pythag_sin_sq(x)
        Real.1 + x.sin * x.sin - x.cos * x.cos = two * (x.sin * x.sin)
        half_pythag_then_sin(x)
        (Real.1 / two) * (Real.1 + x.sin * x.sin - x.cos * x.cos) =
            x.sin * x.sin
        pointwise_mul(constant[Real, Real](Real.1 / two),
            pointwise_add(constant[Real, Real](Real.1),
                pointwise_neg(pointwise_add(pointwise_mul(Real.sin, pointwise_neg(Real.sin)), pointwise_mul(Real.cos, Real.cos)))), x) =
            x.sin * x.sin
        fn_sin_sq(x) = x.sin * x.sin
        pointwise_mul(constant[Real, Real](Real.1 / two),
            pointwise_add(constant[Real, Real](Real.1),
                pointwise_neg(pointwise_add(pointwise_mul(Real.sin, pointwise_neg(Real.sin)), pointwise_mul(Real.cos, Real.cos)))), x) =
            fn_sin_sq(x)
    }
    function_extensionality(pointwise_mul(constant[Real, Real](Real.1 / two),
        pointwise_add(constant[Real, Real](Real.1),
            pointwise_neg(pointwise_add(pointwise_mul(Real.sin, pointwise_neg(Real.sin)), pointwise_mul(Real.cos, Real.cos))))),
        fn_sin_sq)
    pointwise_mul(constant[Real, Real](Real.1 / two),
        pointwise_add(constant[Real, Real](Real.1),
            pointwise_neg(pointwise_add(pointwise_mul(Real.sin, pointwise_neg(Real.sin)), pointwise_mul(Real.cos, Real.cos))))) =
        fn_sin_sq
    is_derivative_fn(g_sin_sq, fn_sin_sq)
}

/// The derivative of g_cos_sq is fn_cos_sq.
theorem g_cos_sq_has_derivative {
    is_derivative_fn(g_cos_sq, fn_cos_sq)
} by {
    sin_is_derivative_fn
    is_derivative_fn(Real.sin, Real.cos)
    cos_derivative_is_neg_sin
    is_derivative_fn(Real.cos, pointwise_neg(Real.sin))
    derivative_fn_mul(Real.sin, Real.cos, Real.cos, pointwise_neg(Real.sin))
    is_derivative_fn(pointwise_mul(Real.sin, Real.cos),
        pointwise_add(pointwise_mul(Real.sin, pointwise_neg(Real.sin)), pointwise_mul(Real.cos, Real.cos)))
    derivative_fn_identity
    is_derivative_fn(identity_fn[Real], constant[Real, Real](Real.1))
    derivative_fn_add(identity_fn[Real], pointwise_mul(Real.sin, Real.cos),
        constant[Real, Real](Real.1),
        pointwise_add(pointwise_mul(Real.sin, pointwise_neg(Real.sin)), pointwise_mul(Real.cos, Real.cos)))
    is_derivative_fn(pointwise_add(identity_fn[Real], pointwise_mul(Real.sin, Real.cos)),
        pointwise_add(constant[Real, Real](Real.1),
            pointwise_add(pointwise_mul(Real.sin, pointwise_neg(Real.sin)), pointwise_mul(Real.cos, Real.cos))))
    derivative_fn_const_mul(Real.1 / two,
        pointwise_add(identity_fn[Real], pointwise_mul(Real.sin, Real.cos)),
        pointwise_add(constant[Real, Real](Real.1),
            pointwise_add(pointwise_mul(Real.sin, pointwise_neg(Real.sin)), pointwise_mul(Real.cos, Real.cos))))
    is_derivative_fn(g_cos_sq,
        pointwise_mul(constant[Real, Real](Real.1 / two),
            pointwise_add(constant[Real, Real](Real.1),
                pointwise_add(pointwise_mul(Real.sin, pointwise_neg(Real.sin)), pointwise_mul(Real.cos, Real.cos)))))
    forall(x: Real) {
        pointwise_mul_apply(constant[Real, Real](Real.1 / two),
            pointwise_add(constant[Real, Real](Real.1),
                pointwise_add(pointwise_mul(Real.sin, pointwise_neg(Real.sin)), pointwise_mul(Real.cos, Real.cos))), x)
        pointwise_mul(constant[Real, Real](Real.1 / two),
            pointwise_add(constant[Real, Real](Real.1),
                pointwise_add(pointwise_mul(Real.sin, pointwise_neg(Real.sin)), pointwise_mul(Real.cos, Real.cos))), x) =
            constant[Real, Real](Real.1 / two, x) * pointwise_add(constant[Real, Real](Real.1),
                pointwise_add(pointwise_mul(Real.sin, pointwise_neg(Real.sin)), pointwise_mul(Real.cos, Real.cos)), x)
        constant[Real, Real](Real.1 / two, x) = Real.1 / two
        pointwise_add_apply(constant[Real, Real](Real.1),
            pointwise_add(pointwise_mul(Real.sin, pointwise_neg(Real.sin)), pointwise_mul(Real.cos, Real.cos)), x)
        pointwise_add(constant[Real, Real](Real.1),
            pointwise_add(pointwise_mul(Real.sin, pointwise_neg(Real.sin)), pointwise_mul(Real.cos, Real.cos)), x) =
            constant[Real, Real](Real.1, x) + pointwise_add(pointwise_mul(Real.sin, pointwise_neg(Real.sin)), pointwise_mul(Real.cos, Real.cos), x)
        constant[Real, Real](Real.1, x) = Real.1
        pointwise_add_apply(pointwise_mul(Real.sin, pointwise_neg(Real.sin)), pointwise_mul(Real.cos, Real.cos), x)
        pointwise_add(pointwise_mul(Real.sin, pointwise_neg(Real.sin)), pointwise_mul(Real.cos, Real.cos), x) =
            pointwise_mul(Real.sin, pointwise_neg(Real.sin), x) + pointwise_mul(Real.cos, Real.cos, x)
        pointwise_mul_apply(Real.sin, pointwise_neg(Real.sin), x)
        pointwise_mul(Real.sin, pointwise_neg(Real.sin), x) = x.sin * pointwise_neg(Real.sin, x)
        pointwise_neg_apply(Real.sin, x)
        pointwise_neg(Real.sin, x) = -x.sin
        pointwise_mul_apply(Real.cos, Real.cos, x)
        pointwise_mul(Real.cos, Real.cos, x) = x.cos * x.cos
        pointwise_mul(constant[Real, Real](Real.1 / two),
            pointwise_add(constant[Real, Real](Real.1),
                pointwise_add(pointwise_mul(Real.sin, pointwise_neg(Real.sin)), pointwise_mul(Real.cos, Real.cos))), x) =
            (Real.1 / two) * (Real.1 + (x.sin * pointwise_neg(Real.sin, x) + x.cos * x.cos))
        pointwise_neg(Real.sin, x) = -x.sin
        (Real.1 / two) * (Real.1 + (x.sin * pointwise_neg(Real.sin, x) + x.cos * x.cos)) =
            (Real.1 / two) * (Real.1 + (x.sin * -x.sin + x.cos * x.cos))
        x.sin * -x.sin = -(x.sin * x.sin)
        (Real.1 / two) * (Real.1 + (x.sin * -x.sin + x.cos * x.cos)) =
            (Real.1 / two) * (Real.1 - x.sin * x.sin + x.cos * x.cos)
        half_pythag_cos_sq(x)
        Real.1 - x.sin * x.sin + x.cos * x.cos = two * (x.cos * x.cos)
        half_pythag_then_cos(x)
        (Real.1 / two) * (Real.1 - x.sin * x.sin + x.cos * x.cos) =
            x.cos * x.cos
        pointwise_mul(constant[Real, Real](Real.1 / two),
            pointwise_add(constant[Real, Real](Real.1),
                pointwise_add(pointwise_mul(Real.sin, pointwise_neg(Real.sin)), pointwise_mul(Real.cos, Real.cos))), x) =
            x.cos * x.cos
        fn_cos_sq(x) = x.cos * x.cos
        pointwise_mul(constant[Real, Real](Real.1 / two),
            pointwise_add(constant[Real, Real](Real.1),
                pointwise_add(pointwise_mul(Real.sin, pointwise_neg(Real.sin)), pointwise_mul(Real.cos, Real.cos))), x) =
            fn_cos_sq(x)
    }
    function_extensionality(pointwise_mul(constant[Real, Real](Real.1 / two),
        pointwise_add(constant[Real, Real](Real.1),
            pointwise_add(pointwise_mul(Real.sin, pointwise_neg(Real.sin)), pointwise_mul(Real.cos, Real.cos)))),
        fn_cos_sq)
    pointwise_mul(constant[Real, Real](Real.1 / two),
        pointwise_add(constant[Real, Real](Real.1),
            pointwise_add(pointwise_mul(Real.sin, pointwise_neg(Real.sin)), pointwise_mul(Real.cos, Real.cos)))) =
        fn_cos_sq
    is_derivative_fn(g_cos_sq, fn_cos_sq)
}

/// The derivative of g_sin_sin2 is fn_sin_sin2.
theorem g_sin_sin2_has_derivative {
    is_derivative_fn(g_sin_sin2, fn_sin_sin2)
} by {
    sin_is_derivative_fn
    is_derivative_fn(Real.sin, Real.cos)
    derivative_fn_square(Real.sin, Real.cos)
    is_derivative_fn(pointwise_mul(Real.sin, Real.sin),
        pointwise_add(pointwise_mul(Real.sin, Real.cos), pointwise_mul(Real.sin, Real.cos)))
    derivative_fn_mul(pointwise_mul(Real.sin, Real.sin), Real.sin,
        pointwise_add(pointwise_mul(Real.sin, Real.cos), pointwise_mul(Real.sin, Real.cos)), Real.cos)
    is_derivative_fn(pointwise_mul(pointwise_mul(Real.sin, Real.sin), Real.sin),
        pointwise_add(pointwise_mul(pointwise_mul(Real.sin, Real.sin), Real.cos),
            pointwise_mul(Real.sin, pointwise_add(pointwise_mul(Real.sin, Real.cos), pointwise_mul(Real.sin, Real.cos)))))
    derivative_fn_const_mul(two / from_nat[Real](Nat.3), pointwise_mul(pointwise_mul(Real.sin, Real.sin), Real.sin),
        pointwise_add(pointwise_mul(pointwise_mul(Real.sin, Real.sin), Real.cos),
            pointwise_mul(Real.sin, pointwise_add(pointwise_mul(Real.sin, Real.cos), pointwise_mul(Real.sin, Real.cos)))))
    is_derivative_fn(g_sin_sin2,
        pointwise_mul(constant[Real, Real](two / from_nat[Real](Nat.3)),
            pointwise_add(pointwise_mul(pointwise_mul(Real.sin, Real.sin), Real.cos),
                pointwise_mul(Real.sin, pointwise_add(pointwise_mul(Real.sin, Real.cos), pointwise_mul(Real.sin, Real.cos))))))
    forall(x: Real) {
        pointwise_mul_apply(constant[Real, Real](two / from_nat[Real](Nat.3)),
            pointwise_add(pointwise_mul(pointwise_mul(Real.sin, Real.sin), Real.cos),
                pointwise_mul(Real.sin, pointwise_add(pointwise_mul(Real.sin, Real.cos), pointwise_mul(Real.sin, Real.cos)))), x)
        pointwise_mul(constant[Real, Real](two / from_nat[Real](Nat.3)),
            pointwise_add(pointwise_mul(pointwise_mul(Real.sin, Real.sin), Real.cos),
                pointwise_mul(Real.sin, pointwise_add(pointwise_mul(Real.sin, Real.cos), pointwise_mul(Real.sin, Real.cos)))), x) =
            constant[Real, Real](two / from_nat[Real](Nat.3), x) *
                pointwise_add(pointwise_mul(pointwise_mul(Real.sin, Real.sin), Real.cos),
                    pointwise_mul(Real.sin, pointwise_add(pointwise_mul(Real.sin, Real.cos), pointwise_mul(Real.sin, Real.cos))), x)
        constant[Real, Real](two / from_nat[Real](Nat.3), x) = two / from_nat[Real](Nat.3)
        pointwise_add_apply(pointwise_mul(pointwise_mul(Real.sin, Real.sin), Real.cos),
            pointwise_mul(Real.sin, pointwise_add(pointwise_mul(Real.sin, Real.cos), pointwise_mul(Real.sin, Real.cos))), x)
        pointwise_add(pointwise_mul(pointwise_mul(Real.sin, Real.sin), Real.cos),
            pointwise_mul(Real.sin, pointwise_add(pointwise_mul(Real.sin, Real.cos), pointwise_mul(Real.sin, Real.cos))), x) =
            pointwise_mul(pointwise_mul(Real.sin, Real.sin), Real.cos, x) +
                pointwise_mul(Real.sin, pointwise_add(pointwise_mul(Real.sin, Real.cos), pointwise_mul(Real.sin, Real.cos)), x)
        pointwise_mul_apply(pointwise_mul(Real.sin, Real.sin), Real.cos, x)
        pointwise_mul(pointwise_mul(Real.sin, Real.sin), Real.cos, x) = pointwise_mul(Real.sin, Real.sin, x) * x.cos
        pointwise_mul_apply(Real.sin, Real.sin, x)
        pointwise_mul(Real.sin, Real.sin, x) = x.sin * x.sin
        pointwise_mul_apply(Real.sin, pointwise_add(pointwise_mul(Real.sin, Real.cos), pointwise_mul(Real.sin, Real.cos)), x)
        pointwise_mul(Real.sin, pointwise_add(pointwise_mul(Real.sin, Real.cos), pointwise_mul(Real.sin, Real.cos)), x) =
            x.sin * pointwise_add(pointwise_mul(Real.sin, Real.cos), pointwise_mul(Real.sin, Real.cos), x)
        pointwise_add_apply(pointwise_mul(Real.sin, Real.cos), pointwise_mul(Real.sin, Real.cos), x)
        pointwise_add(pointwise_mul(Real.sin, Real.cos), pointwise_mul(Real.sin, Real.cos), x) =
            pointwise_mul(Real.sin, Real.cos, x) + pointwise_mul(Real.sin, Real.cos, x)
        pointwise_mul_apply(Real.sin, Real.cos, x)
        pointwise_mul(Real.sin, Real.cos, x) = x.sin * x.cos
        pointwise_mul(constant[Real, Real](two / from_nat[Real](Nat.3)),
            pointwise_add(pointwise_mul(pointwise_mul(Real.sin, Real.sin), Real.cos),
                pointwise_mul(Real.sin, pointwise_add(pointwise_mul(Real.sin, Real.cos), pointwise_mul(Real.sin, Real.cos)))), x) =
            (two / from_nat[Real](Nat.3)) * (x.sin * x.sin * x.cos +
                x.sin * (x.sin * x.cos + x.sin * x.cos))
        x.sin * (x.sin * x.cos + x.sin * x.cos) =
            two * (x.sin * x.sin * x.cos)
        one_add_two_is_three
        Real.1 + two = from_nat[Real](Nat.3)
        mul_distrib_left(Real.1, two, x.sin * x.sin * x.cos)
        (Real.1 + two) * (x.sin * x.sin * x.cos) =
            Real.1 * (x.sin * x.sin * x.cos) + two * (x.sin * x.sin * x.cos)
        Real.1 * (x.sin * x.sin * x.cos) = x.sin * x.sin * x.cos
        from_nat[Real](Nat.3) * (x.sin * x.sin * x.cos) =
            x.sin * x.sin * x.cos + two * (x.sin * x.sin * x.cos)
        (two / from_nat[Real](Nat.3)) * (x.sin * x.sin * x.cos + x.sin * (x.sin * x.cos + x.sin * x.cos)) =
            (two / from_nat[Real](Nat.3)) * (from_nat[Real](Nat.3) * (x.sin * x.sin * x.cos))
        from_nat[Real](Nat.3) != Real.0
        mul_div_cancel(two, from_nat[Real](Nat.3))
        from_nat[Real](Nat.3) != Real.0 implies from_nat[Real](Nat.3) * (two / from_nat[Real](Nat.3)) = two
        from_nat[Real](Nat.3) * (two / from_nat[Real](Nat.3)) = two
        (two / from_nat[Real](Nat.3)) * from_nat[Real](Nat.3) = two
        (two / from_nat[Real](Nat.3)) * (from_nat[Real](Nat.3) * (x.sin * x.sin * x.cos)) =
            two * (x.sin * x.sin * x.cos)
        (two / from_nat[Real](Nat.3)) * (x.sin * x.sin * x.cos + x.sin * (x.sin * x.cos + x.sin * x.cos)) =
            two * (x.sin * x.sin * x.cos)
        sin_double(x)
        (two * x).sin = two * x.sin * x.cos
        fn_sin_sin2(x) = x.sin * (two * x).sin
        fn_sin_sin2(x) = x.sin * (two * x.sin * x.cos)
        x.sin * (two * x.sin * x.cos) = two * (x.sin * x.sin * x.cos)
        fn_sin_sin2(x) = two * (x.sin * x.sin * x.cos)
        pointwise_mul(constant[Real, Real](two / from_nat[Real](Nat.3)),
            pointwise_add(pointwise_mul(pointwise_mul(Real.sin, Real.sin), Real.cos),
                pointwise_mul(Real.sin, pointwise_add(pointwise_mul(Real.sin, Real.cos), pointwise_mul(Real.sin, Real.cos)))), x) =
            fn_sin_sin2(x)
    }
    function_extensionality(pointwise_mul(constant[Real, Real](two / from_nat[Real](Nat.3)),
        pointwise_add(pointwise_mul(pointwise_mul(Real.sin, Real.sin), Real.cos),
            pointwise_mul(Real.sin, pointwise_add(pointwise_mul(Real.sin, Real.cos), pointwise_mul(Real.sin, Real.cos))))),
        fn_sin_sin2)
    pointwise_mul(constant[Real, Real](two / from_nat[Real](Nat.3)),
        pointwise_add(pointwise_mul(pointwise_mul(Real.sin, Real.sin), Real.cos),
            pointwise_mul(Real.sin, pointwise_add(pointwise_mul(Real.sin, Real.cos), pointwise_mul(Real.sin, Real.cos))))) =
        fn_sin_sin2
    is_derivative_fn(g_sin_sin2, fn_sin_sin2)
}

/// The derivative of g_x_sin is fn_x_sin.
theorem g_x_sin_has_derivative {
    is_derivative_fn(g_x_sin, fn_x_sin)
} by {
    sin_is_derivative_fn
    is_derivative_fn(Real.sin, Real.cos)
    cos_derivative_is_neg_sin
    is_derivative_fn(Real.cos, pointwise_neg(Real.sin))
    derivative_fn_identity
    is_derivative_fn(identity_fn[Real], constant[Real, Real](Real.1))
    derivative_fn_mul(identity_fn[Real], Real.cos,
        constant[Real, Real](Real.1), pointwise_neg(Real.sin))
    is_derivative_fn(pointwise_mul(identity_fn[Real], Real.cos),
        pointwise_add(pointwise_mul(identity_fn[Real], pointwise_neg(Real.sin)),
            pointwise_mul(Real.cos, constant[Real, Real](Real.1))))
    derivative_fn_neg(pointwise_mul(identity_fn[Real], Real.cos),
        pointwise_add(pointwise_mul(identity_fn[Real], pointwise_neg(Real.sin)),
            pointwise_mul(Real.cos, constant[Real, Real](Real.1))))
    is_derivative_fn(pointwise_neg(pointwise_mul(identity_fn[Real], Real.cos)),
        pointwise_neg(pointwise_add(pointwise_mul(identity_fn[Real], pointwise_neg(Real.sin)),
            pointwise_mul(Real.cos, constant[Real, Real](Real.1)))))
    derivative_fn_add(Real.sin, pointwise_neg(pointwise_mul(identity_fn[Real], Real.cos)),
        Real.cos,
        pointwise_neg(pointwise_add(pointwise_mul(identity_fn[Real], pointwise_neg(Real.sin)),
            pointwise_mul(Real.cos, constant[Real, Real](Real.1)))))
    is_derivative_fn(pointwise_add(Real.sin, pointwise_neg(pointwise_mul(identity_fn[Real], Real.cos))),
        pointwise_add(Real.cos,
            pointwise_neg(pointwise_add(pointwise_mul(identity_fn[Real], pointwise_neg(Real.sin)),
                pointwise_mul(Real.cos, constant[Real, Real](Real.1))))))
    forall(x: Real) {
        pointwise_add_apply(Real.cos,
            pointwise_neg(pointwise_add(pointwise_mul(identity_fn[Real], pointwise_neg(Real.sin)),
                pointwise_mul(Real.cos, constant[Real, Real](Real.1)))), x)
        pointwise_add(Real.cos,
            pointwise_neg(pointwise_add(pointwise_mul(identity_fn[Real], pointwise_neg(Real.sin)),
                pointwise_mul(Real.cos, constant[Real, Real](Real.1)))), x) =
            x.cos + pointwise_neg(pointwise_add(pointwise_mul(identity_fn[Real], pointwise_neg(Real.sin)),
                pointwise_mul(Real.cos, constant[Real, Real](Real.1))), x)
        pointwise_neg_apply(pointwise_add(pointwise_mul(identity_fn[Real], pointwise_neg(Real.sin)),
            pointwise_mul(Real.cos, constant[Real, Real](Real.1))), x)
        pointwise_neg(pointwise_add(pointwise_mul(identity_fn[Real], pointwise_neg(Real.sin)),
            pointwise_mul(Real.cos, constant[Real, Real](Real.1))), x) =
            -(pointwise_add(pointwise_mul(identity_fn[Real], pointwise_neg(Real.sin)),
                pointwise_mul(Real.cos, constant[Real, Real](Real.1)), x))
        pointwise_add_apply(pointwise_mul(identity_fn[Real], pointwise_neg(Real.sin)),
            pointwise_mul(Real.cos, constant[Real, Real](Real.1)), x)
        pointwise_add(pointwise_mul(identity_fn[Real], pointwise_neg(Real.sin)),
            pointwise_mul(Real.cos, constant[Real, Real](Real.1)), x) =
            pointwise_mul(identity_fn[Real], pointwise_neg(Real.sin), x) +
                pointwise_mul(Real.cos, constant[Real, Real](Real.1), x)
        pointwise_mul_apply(identity_fn[Real], pointwise_neg(Real.sin), x)
        pointwise_mul(identity_fn[Real], pointwise_neg(Real.sin), x) = identity_fn[Real](x) * pointwise_neg(Real.sin, x)
        pointwise_neg_apply(Real.sin, x)
        pointwise_neg(Real.sin, x) = -x.sin
        pointwise_mul_apply(Real.cos, constant[Real, Real](Real.1), x)
        pointwise_mul(Real.cos, constant[Real, Real](Real.1), x) = x.cos * constant[Real, Real](Real.1, x)
        pointwise_add(Real.cos,
            pointwise_neg(pointwise_add(pointwise_mul(identity_fn[Real], pointwise_neg(Real.sin)),
                pointwise_mul(Real.cos, constant[Real, Real](Real.1)))), x) =
            x.cos + -(identity_fn[Real](x) * pointwise_neg(Real.sin, x) + x.cos * constant[Real, Real](Real.1, x))
        identity_fn[Real](x) = x
        pointwise_neg(Real.sin, x) = -x.sin
        constant[Real, Real](Real.1, x) = Real.1
        x.cos + -(identity_fn[Real](x) * pointwise_neg(Real.sin, x) + x.cos * constant[Real, Real](Real.1, x)) =
            x.cos + -(x * -x.sin + x.cos * Real.1)
        x * -x.sin = -(x * x.sin)
        x.cos * Real.1 = x.cos
        x.cos + -(x * -x.sin + x.cos * Real.1) = x.cos + -(-(x * x.sin) + x.cos)
        neg_distrib(-(x * x.sin), x.cos)
        -(-(x * x.sin) + x.cos) = -(-(x * x.sin)) + -x.cos
        neg_neg(x * x.sin)
        -(-(x * x.sin)) = x * x.sin
        -(-(x * x.sin) + x.cos) = x * x.sin + -x.cos
        x.cos + -(-(x * x.sin) + x.cos) = x.cos + (x * x.sin + -x.cos)
        add_assoc(x.cos, x * x.sin, -x.cos)
        (x.cos + x * x.sin) + -x.cos = x.cos + (x * x.sin + -x.cos)
        add_comm(x.cos, x * x.sin)
        x.cos + x * x.sin = x * x.sin + x.cos
        (x * x.sin + x.cos) + -x.cos = x.cos + (x * x.sin + -x.cos)
        add_assoc(x * x.sin, x.cos, -x.cos)
        (x * x.sin + x.cos) + -x.cos = x * x.sin + (x.cos + -x.cos)
        add_neg_eq_zero(x.cos)
        x.cos + -x.cos = Real.0
        x * x.sin + (x.cos + -x.cos) = x * x.sin + Real.0
        add_zero_right(x * x.sin)
        x * x.sin + Real.0 = x * x.sin
        (x * x.sin + x.cos) + -x.cos = x * x.sin
        x.cos + (x * x.sin + -x.cos) = x * x.sin
        x.cos + -(-(x * x.sin) + x.cos) = x * x.sin
        pointwise_add(Real.cos,
            pointwise_neg(pointwise_add(pointwise_mul(identity_fn[Real], pointwise_neg(Real.sin)),
                pointwise_mul(Real.cos, constant[Real, Real](Real.1)))), x) = x * x.sin
        fn_x_sin(x) = identity_fn[Real](x) * x.sin
        identity_fn[Real](x) = x
        fn_x_sin(x) = x * x.sin
        pointwise_add(Real.cos,
            pointwise_neg(pointwise_add(pointwise_mul(identity_fn[Real], pointwise_neg(Real.sin)),
                pointwise_mul(Real.cos, constant[Real, Real](Real.1)))), x) = fn_x_sin(x)
    }
    function_extensionality(pointwise_add(Real.cos,
        pointwise_neg(pointwise_add(pointwise_mul(identity_fn[Real], pointwise_neg(Real.sin)),
            pointwise_mul(Real.cos, constant[Real, Real](Real.1))))),
        fn_x_sin)
    pointwise_add(Real.cos,
        pointwise_neg(pointwise_add(pointwise_mul(identity_fn[Real], pointwise_neg(Real.sin)),
            pointwise_mul(Real.cos, constant[Real, Real](Real.1))))) = fn_x_sin
    is_derivative_fn(g_x_sin, fn_x_sin)
}

// ---------------------------------------------------------------------------
// Continuity of the antiderivatives
// ---------------------------------------------------------------------------

/// The antiderivative of (x).sincos(x) is continuous.
theorem g_sincos_continuous {
    continuous(g_sincos)
} by {
    sin_continuous
    continuous(Real.sin)
    continuous_pointwise_mul(Real.sin, Real.sin)
    continuous(pointwise_mul(Real.sin, Real.sin))
    constant_function_is_continuous(Real.1 / two)
    continuous(constant[Real, Real](Real.1 / two))
    continuous_pointwise_mul(constant[Real, Real](Real.1 / two), pointwise_mul(Real.sin, Real.sin))
    continuous(pointwise_mul(constant[Real, Real](Real.1 / two), pointwise_mul(Real.sin, Real.sin)))
    continuous(g_sincos)
}

/// The antiderivative of x.sin^2 is continuous.
theorem g_sin_sq_continuous {
    continuous(g_sin_sq)
} by {
    sin_continuous
    continuous(Real.sin)
    cos_continuous
    continuous(Real.cos)
    continuous_pointwise_mul(Real.sin, Real.cos)
    continuous(pointwise_mul(Real.sin, Real.cos))
    continuous_pointwise_neg(pointwise_mul(Real.sin, Real.cos))
    continuous(pointwise_neg(pointwise_mul(Real.sin, Real.cos)))
    identity_function_is_continuous
    continuous(identity_fn[Real])
    continuous_pointwise_add(identity_fn[Real], pointwise_neg(pointwise_mul(Real.sin, Real.cos)))
    continuous(pointwise_add(identity_fn[Real], pointwise_neg(pointwise_mul(Real.sin, Real.cos))))
    constant_function_is_continuous(Real.1 / two)
    continuous(constant[Real, Real](Real.1 / two))
    continuous_pointwise_mul(constant[Real, Real](Real.1 / two),
        pointwise_add(identity_fn[Real], pointwise_neg(pointwise_mul(Real.sin, Real.cos))))
    continuous(pointwise_mul(constant[Real, Real](Real.1 / two),
        pointwise_add(identity_fn[Real], pointwise_neg(pointwise_mul(Real.sin, Real.cos)))))
    continuous(g_sin_sq)
}

/// The antiderivative of x.cos^2 is continuous.
theorem g_cos_sq_continuous {
    continuous(g_cos_sq)
} by {
    sin_continuous
    continuous(Real.sin)
    cos_continuous
    continuous(Real.cos)
    continuous_pointwise_mul(Real.sin, Real.cos)
    continuous(pointwise_mul(Real.sin, Real.cos))
    identity_function_is_continuous
    continuous(identity_fn[Real])
    continuous_pointwise_add(identity_fn[Real], pointwise_mul(Real.sin, Real.cos))
    continuous(pointwise_add(identity_fn[Real], pointwise_mul(Real.sin, Real.cos)))
    constant_function_is_continuous(Real.1 / two)
    continuous(constant[Real, Real](Real.1 / two))
    continuous_pointwise_mul(constant[Real, Real](Real.1 / two),
        pointwise_add(identity_fn[Real], pointwise_mul(Real.sin, Real.cos)))
    continuous(pointwise_mul(constant[Real, Real](Real.1 / two),
        pointwise_add(identity_fn[Real], pointwise_mul(Real.sin, Real.cos))))
    continuous(g_cos_sq)
}

/// The antiderivative of x.sin(2x).sin is continuous.
theorem g_sin_sin2_continuous {
    continuous(g_sin_sin2)
} by {
    sin_continuous
    continuous(Real.sin)
    continuous_pointwise_mul(Real.sin, Real.sin)
    continuous(pointwise_mul(Real.sin, Real.sin))
    continuous_pointwise_mul(pointwise_mul(Real.sin, Real.sin), Real.sin)
    continuous(pointwise_mul(pointwise_mul(Real.sin, Real.sin), Real.sin))
    constant_function_is_continuous(two / from_nat[Real](Nat.3))
    continuous(constant[Real, Real](two / from_nat[Real](Nat.3)))
    continuous_pointwise_mul(constant[Real, Real](two / from_nat[Real](Nat.3)),
        pointwise_mul(pointwise_mul(Real.sin, Real.sin), Real.sin))
    continuous(pointwise_mul(constant[Real, Real](two / from_nat[Real](Nat.3)),
        pointwise_mul(pointwise_mul(Real.sin, Real.sin), Real.sin)))
    continuous(g_sin_sin2)
}

/// The antiderivative of x x.sin is continuous.
theorem g_x_sin_continuous {
    continuous(g_x_sin)
} by {
    sin_continuous
    continuous(Real.sin)
    cos_continuous
    continuous(Real.cos)
    identity_function_is_continuous
    continuous(identity_fn[Real])
    continuous_pointwise_mul(identity_fn[Real], Real.cos)
    continuous(pointwise_mul(identity_fn[Real], Real.cos))
    continuous_pointwise_neg(pointwise_mul(identity_fn[Real], Real.cos))
    continuous(pointwise_neg(pointwise_mul(identity_fn[Real], Real.cos)))
    continuous_pointwise_add(Real.sin, pointwise_neg(pointwise_mul(identity_fn[Real], Real.cos)))
    continuous(pointwise_add(Real.sin, pointwise_neg(pointwise_mul(identity_fn[Real], Real.cos))))
    continuous(g_x_sin)
}



// ---------------------------------------------------------------------------
// Integrability of the product functions on [0, 2pi]
// ---------------------------------------------------------------------------

/// fn_sincos is integrable on [0, 2pi].
theorem fn_sincos_integrable {
    is_integrable(fn_sincos, Real.0, two_pi)
} by {
    two_pi_nonneg
    Real.0 <= two_pi
        two_positive
    two.is_positive
    pos_gt_zero(two)
    two > Real.0
    lt_imp_lte(Real.0, two)
    Real.0 <= two
    g_sincos_continuous
    continuous(g_sincos)
    g_sincos_has_derivative
    is_derivative_fn(g_sincos, fn_sincos)
    forall(u: Real, v: Real) {
        if interval_contains(Real.0, two_pi, u) and interval_contains(Real.0, two_pi, v) {
            fn_sincos_lipschitz_m2(u, v)
            (fn_sincos(u) - fn_sincos(v)).abs <= two * (u - v).abs
        }
    }
    forall(t: Real) {
        if interval_contains(Real.0, two_pi, t) {
            fn_sincos_abs_le_one(t)
            fn_sincos(t).abs <= Real.1
            abs_le_one_imp_lower(fn_sincos(t))
            -Real.1 <= fn_sincos(t)
        }
    }
    forall(t: Real) {
        if interval_contains(Real.0, two_pi, t) {
            fn_sincos_abs_le_one(t)
            fn_sincos(t).abs <= Real.1
            abs_le_one_imp_upper(fn_sincos(t))
            fn_sincos(t) <= Real.1
        }
    }
        Real.0 <= two_pi
    (Real.0 <= two_pi) and Real.0 <= two
    ((Real.0 <= two_pi) and Real.0 <= two) and continuous(g_sincos)
    (((Real.0 <= two_pi) and Real.0 <= two) and continuous(g_sincos)) and is_derivative_fn(g_sincos, fn_sincos)
    ((((Real.0 <= two_pi) and Real.0 <= two) and continuous(g_sincos)) and is_derivative_fn(g_sincos, fn_sincos)) and forall(u: Real, v: Real) {
            interval_contains(Real.0, two_pi, u) and interval_contains(Real.0, two_pi, v)
            implies (fn_sincos(u) - fn_sincos(v)).abs <= two * (u - v).abs
        }
    (((((Real.0 <= two_pi) and Real.0 <= two) and continuous(g_sincos)) and is_derivative_fn(g_sincos, fn_sincos)) and forall(u: Real, v: Real) {
            interval_contains(Real.0, two_pi, u) and interval_contains(Real.0, two_pi, v)
            implies (fn_sincos(u) - fn_sincos(v)).abs <= two * (u - v).abs
        }) and forall(t: Real) { interval_contains(Real.0, two_pi, t) implies -Real.1 <= fn_sincos(t) }
    ((((((Real.0 <= two_pi) and Real.0 <= two) and continuous(g_sincos)) and is_derivative_fn(g_sincos, fn_sincos)) and forall(u: Real, v: Real) {
            interval_contains(Real.0, two_pi, u) and interval_contains(Real.0, two_pi, v)
            implies (fn_sincos(u) - fn_sincos(v)).abs <= two * (u - v).abs
        }) and forall(t: Real) { interval_contains(Real.0, two_pi, t) implies -Real.1 <= fn_sincos(t) }) and forall(t: Real) { interval_contains(Real.0, two_pi, t) implies fn_sincos(t) <= Real.1 }
    if ((((((Real.0 <= two_pi) and Real.0 <= two) and continuous(g_sincos)) and is_derivative_fn(g_sincos, fn_sincos)) and forall(u: Real, v: Real) {
            interval_contains(Real.0, two_pi, u) and interval_contains(Real.0, two_pi, v)
            implies (fn_sincos(u) - fn_sincos(v)).abs <= two * (u - v).abs
        }) and forall(t: Real) { interval_contains(Real.0, two_pi, t) implies -Real.1 <= fn_sincos(t) }) and forall(t: Real) { interval_contains(Real.0, two_pi, t) implies fn_sincos(t) <= Real.1 } {
        fn_integrable_gen(fn_sincos, g_sincos, Real.0, two_pi, two, -Real.1, Real.1)
        is_integrable(fn_sincos, Real.0, two_pi)
    }
}
/// fn_sin_sq is integrable on [0, 2pi].
theorem fn_sin_sq_integrable {
    is_integrable(fn_sin_sq, Real.0, two_pi)
} by {
    two_pi_nonneg
    Real.0 <= two_pi
        two_positive
    two.is_positive
    pos_gt_zero(two)
    two > Real.0
    lt_imp_lte(Real.0, two)
    Real.0 <= two
    g_sin_sq_continuous
    continuous(g_sin_sq)
    g_sin_sq_has_derivative
    is_derivative_fn(g_sin_sq, fn_sin_sq)
    forall(u: Real, v: Real) {
        if interval_contains(Real.0, two_pi, u) and interval_contains(Real.0, two_pi, v) {
            fn_sin_sq_lipschitz_m2(u, v)
            (fn_sin_sq(u) - fn_sin_sq(v)).abs <= two * (u - v).abs
        }
    }
    forall(t: Real) {
        if interval_contains(Real.0, two_pi, t) {
            fn_sin_sq_abs_le_one(t)
            fn_sin_sq(t).abs <= Real.1
            abs_le_one_imp_lower(fn_sin_sq(t))
            -Real.1 <= fn_sin_sq(t)
        }
    }
    forall(t: Real) {
        if interval_contains(Real.0, two_pi, t) {
            fn_sin_sq_abs_le_one(t)
            fn_sin_sq(t).abs <= Real.1
            abs_le_one_imp_upper(fn_sin_sq(t))
            fn_sin_sq(t) <= Real.1
        }
    }
        Real.0 <= two_pi
    (Real.0 <= two_pi) and Real.0 <= two
    ((Real.0 <= two_pi) and Real.0 <= two) and continuous(g_sin_sq)
    (((Real.0 <= two_pi) and Real.0 <= two) and continuous(g_sin_sq)) and is_derivative_fn(g_sin_sq, fn_sin_sq)
    ((((Real.0 <= two_pi) and Real.0 <= two) and continuous(g_sin_sq)) and is_derivative_fn(g_sin_sq, fn_sin_sq)) and forall(u: Real, v: Real) {
            interval_contains(Real.0, two_pi, u) and interval_contains(Real.0, two_pi, v)
            implies (fn_sin_sq(u) - fn_sin_sq(v)).abs <= two * (u - v).abs
        }
    (((((Real.0 <= two_pi) and Real.0 <= two) and continuous(g_sin_sq)) and is_derivative_fn(g_sin_sq, fn_sin_sq)) and forall(u: Real, v: Real) {
            interval_contains(Real.0, two_pi, u) and interval_contains(Real.0, two_pi, v)
            implies (fn_sin_sq(u) - fn_sin_sq(v)).abs <= two * (u - v).abs
        }) and forall(t: Real) { interval_contains(Real.0, two_pi, t) implies -Real.1 <= fn_sin_sq(t) }
    ((((((Real.0 <= two_pi) and Real.0 <= two) and continuous(g_sin_sq)) and is_derivative_fn(g_sin_sq, fn_sin_sq)) and forall(u: Real, v: Real) {
            interval_contains(Real.0, two_pi, u) and interval_contains(Real.0, two_pi, v)
            implies (fn_sin_sq(u) - fn_sin_sq(v)).abs <= two * (u - v).abs
        }) and forall(t: Real) { interval_contains(Real.0, two_pi, t) implies -Real.1 <= fn_sin_sq(t) }) and forall(t: Real) { interval_contains(Real.0, two_pi, t) implies fn_sin_sq(t) <= Real.1 }
    if ((((((Real.0 <= two_pi) and Real.0 <= two) and continuous(g_sin_sq)) and is_derivative_fn(g_sin_sq, fn_sin_sq)) and forall(u: Real, v: Real) {
            interval_contains(Real.0, two_pi, u) and interval_contains(Real.0, two_pi, v)
            implies (fn_sin_sq(u) - fn_sin_sq(v)).abs <= two * (u - v).abs
        }) and forall(t: Real) { interval_contains(Real.0, two_pi, t) implies -Real.1 <= fn_sin_sq(t) }) and forall(t: Real) { interval_contains(Real.0, two_pi, t) implies fn_sin_sq(t) <= Real.1 } {
        fn_integrable_gen(fn_sin_sq, g_sin_sq, Real.0, two_pi, two, -Real.1, Real.1)
        is_integrable(fn_sin_sq, Real.0, two_pi)
    }
}
/// fn_cos_sq is integrable on [0, 2pi].
theorem fn_cos_sq_integrable {
    is_integrable(fn_cos_sq, Real.0, two_pi)
} by {
    two_pi_nonneg
    Real.0 <= two_pi
        two_positive
    two.is_positive
    pos_gt_zero(two)
    two > Real.0
    lt_imp_lte(Real.0, two)
    Real.0 <= two
    g_cos_sq_continuous
    continuous(g_cos_sq)
    g_cos_sq_has_derivative
    is_derivative_fn(g_cos_sq, fn_cos_sq)
    forall(u: Real, v: Real) {
        if interval_contains(Real.0, two_pi, u) and interval_contains(Real.0, two_pi, v) {
            fn_cos_sq_lipschitz_m2(u, v)
            (fn_cos_sq(u) - fn_cos_sq(v)).abs <= two * (u - v).abs
        }
    }
    forall(t: Real) {
        if interval_contains(Real.0, two_pi, t) {
            fn_cos_sq_abs_le_one(t)
            fn_cos_sq(t).abs <= Real.1
            abs_le_one_imp_lower(fn_cos_sq(t))
            -Real.1 <= fn_cos_sq(t)
        }
    }
    forall(t: Real) {
        if interval_contains(Real.0, two_pi, t) {
            fn_cos_sq_abs_le_one(t)
            fn_cos_sq(t).abs <= Real.1
            abs_le_one_imp_upper(fn_cos_sq(t))
            fn_cos_sq(t) <= Real.1
        }
    }
        Real.0 <= two_pi
    (Real.0 <= two_pi) and Real.0 <= two
    ((Real.0 <= two_pi) and Real.0 <= two) and continuous(g_cos_sq)
    (((Real.0 <= two_pi) and Real.0 <= two) and continuous(g_cos_sq)) and is_derivative_fn(g_cos_sq, fn_cos_sq)
    ((((Real.0 <= two_pi) and Real.0 <= two) and continuous(g_cos_sq)) and is_derivative_fn(g_cos_sq, fn_cos_sq)) and forall(u: Real, v: Real) {
            interval_contains(Real.0, two_pi, u) and interval_contains(Real.0, two_pi, v)
            implies (fn_cos_sq(u) - fn_cos_sq(v)).abs <= two * (u - v).abs
        }
    (((((Real.0 <= two_pi) and Real.0 <= two) and continuous(g_cos_sq)) and is_derivative_fn(g_cos_sq, fn_cos_sq)) and forall(u: Real, v: Real) {
            interval_contains(Real.0, two_pi, u) and interval_contains(Real.0, two_pi, v)
            implies (fn_cos_sq(u) - fn_cos_sq(v)).abs <= two * (u - v).abs
        }) and forall(t: Real) { interval_contains(Real.0, two_pi, t) implies -Real.1 <= fn_cos_sq(t) }
    ((((((Real.0 <= two_pi) and Real.0 <= two) and continuous(g_cos_sq)) and is_derivative_fn(g_cos_sq, fn_cos_sq)) and forall(u: Real, v: Real) {
            interval_contains(Real.0, two_pi, u) and interval_contains(Real.0, two_pi, v)
            implies (fn_cos_sq(u) - fn_cos_sq(v)).abs <= two * (u - v).abs
        }) and forall(t: Real) { interval_contains(Real.0, two_pi, t) implies -Real.1 <= fn_cos_sq(t) }) and forall(t: Real) { interval_contains(Real.0, two_pi, t) implies fn_cos_sq(t) <= Real.1 }
    if ((((((Real.0 <= two_pi) and Real.0 <= two) and continuous(g_cos_sq)) and is_derivative_fn(g_cos_sq, fn_cos_sq)) and forall(u: Real, v: Real) {
            interval_contains(Real.0, two_pi, u) and interval_contains(Real.0, two_pi, v)
            implies (fn_cos_sq(u) - fn_cos_sq(v)).abs <= two * (u - v).abs
        }) and forall(t: Real) { interval_contains(Real.0, two_pi, t) implies -Real.1 <= fn_cos_sq(t) }) and forall(t: Real) { interval_contains(Real.0, two_pi, t) implies fn_cos_sq(t) <= Real.1 } {
        fn_integrable_gen(fn_cos_sq, g_cos_sq, Real.0, two_pi, two, -Real.1, Real.1)
        is_integrable(fn_cos_sq, Real.0, two_pi)
    }
}
/// fn_sin_sin2 is integrable on [0, 2pi].
theorem fn_sin_sin2_integrable {
    is_integrable(fn_sin_sin2, Real.0, two_pi)
} by {
    two_pi_nonneg
    Real.0 <= two_pi
        from_nat_suc_pos_real(Nat.2)
    from_nat[Real](Nat.3) > Real.0
    lt_imp_lte(Real.0, from_nat[Real](Nat.3))
    Real.0 <= from_nat[Real](Nat.3)
    g_sin_sin2_continuous
    continuous(g_sin_sin2)
    g_sin_sin2_has_derivative
    is_derivative_fn(g_sin_sin2, fn_sin_sin2)
    forall(u: Real, v: Real) {
        if interval_contains(Real.0, two_pi, u) and interval_contains(Real.0, two_pi, v) {
            fn_sin_sin2_lipschitz_m3(u, v)
            (fn_sin_sin2(u) - fn_sin_sin2(v)).abs <= from_nat[Real](Nat.3) * (u - v).abs
        }
    }
    forall(t: Real) {
        if interval_contains(Real.0, two_pi, t) {
            fn_sin_sin2_abs_le_one(t)
            fn_sin_sin2(t).abs <= Real.1
            abs_le_one_imp_lower(fn_sin_sin2(t))
            -Real.1 <= fn_sin_sin2(t)
        }
    }
    forall(t: Real) {
        if interval_contains(Real.0, two_pi, t) {
            fn_sin_sin2_abs_le_one(t)
            fn_sin_sin2(t).abs <= Real.1
            abs_le_one_imp_upper(fn_sin_sin2(t))
            fn_sin_sin2(t) <= Real.1
        }
    }
        Real.0 <= two_pi
    (Real.0 <= two_pi) and Real.0 <= from_nat[Real](Nat.3)
    ((Real.0 <= two_pi) and Real.0 <= from_nat[Real](Nat.3)) and continuous(g_sin_sin2)
    (((Real.0 <= two_pi) and Real.0 <= from_nat[Real](Nat.3)) and continuous(g_sin_sin2)) and is_derivative_fn(g_sin_sin2, fn_sin_sin2)
    ((((Real.0 <= two_pi) and Real.0 <= from_nat[Real](Nat.3)) and continuous(g_sin_sin2)) and is_derivative_fn(g_sin_sin2, fn_sin_sin2)) and forall(u: Real, v: Real) {
            interval_contains(Real.0, two_pi, u) and interval_contains(Real.0, two_pi, v)
            implies (fn_sin_sin2(u) - fn_sin_sin2(v)).abs <= from_nat[Real](Nat.3) * (u - v).abs
        }
    (((((Real.0 <= two_pi) and Real.0 <= from_nat[Real](Nat.3)) and continuous(g_sin_sin2)) and is_derivative_fn(g_sin_sin2, fn_sin_sin2)) and forall(u: Real, v: Real) {
            interval_contains(Real.0, two_pi, u) and interval_contains(Real.0, two_pi, v)
            implies (fn_sin_sin2(u) - fn_sin_sin2(v)).abs <= from_nat[Real](Nat.3) * (u - v).abs
        }) and forall(t: Real) { interval_contains(Real.0, two_pi, t) implies -Real.1 <= fn_sin_sin2(t) }
    ((((((Real.0 <= two_pi) and Real.0 <= from_nat[Real](Nat.3)) and continuous(g_sin_sin2)) and is_derivative_fn(g_sin_sin2, fn_sin_sin2)) and forall(u: Real, v: Real) {
            interval_contains(Real.0, two_pi, u) and interval_contains(Real.0, two_pi, v)
            implies (fn_sin_sin2(u) - fn_sin_sin2(v)).abs <= from_nat[Real](Nat.3) * (u - v).abs
        }) and forall(t: Real) { interval_contains(Real.0, two_pi, t) implies -Real.1 <= fn_sin_sin2(t) }) and forall(t: Real) { interval_contains(Real.0, two_pi, t) implies fn_sin_sin2(t) <= Real.1 }
    if ((((((Real.0 <= two_pi) and Real.0 <= from_nat[Real](Nat.3)) and continuous(g_sin_sin2)) and is_derivative_fn(g_sin_sin2, fn_sin_sin2)) and forall(u: Real, v: Real) {
            interval_contains(Real.0, two_pi, u) and interval_contains(Real.0, two_pi, v)
            implies (fn_sin_sin2(u) - fn_sin_sin2(v)).abs <= from_nat[Real](Nat.3) * (u - v).abs
        }) and forall(t: Real) { interval_contains(Real.0, two_pi, t) implies -Real.1 <= fn_sin_sin2(t) }) and forall(t: Real) { interval_contains(Real.0, two_pi, t) implies fn_sin_sin2(t) <= Real.1 } {
        fn_integrable_gen(fn_sin_sin2, g_sin_sin2, Real.0, two_pi, from_nat[Real](Nat.3), -Real.1, Real.1)
        is_integrable(fn_sin_sin2, Real.0, two_pi)
    }
}
/// x x.sin is integrable on [0, 2pi].
theorem fn_x_sin_integrable {
    is_integrable(fn_x_sin, Real.0, two_pi)
} by {
    two_pi_nonneg
    Real.0 <= two_pi
    one_nonneg
    Real.0 <= Real.1
    add_le_add(Real.0, Real.1, Real.0, two_pi)
    Real.0 + Real.0 <= Real.1 + two_pi
    Real.0 + Real.0 = Real.0
    Real.0 <= Real.1 + two_pi
    g_x_sin_continuous
    continuous(g_x_sin)
    g_x_sin_has_derivative
    is_derivative_fn(g_x_sin, fn_x_sin)
    forall(u: Real, v: Real) {
        if interval_contains(Real.0, two_pi, u) and interval_contains(Real.0, two_pi, v) {
            fn_x_sin_lipschitz_on(u, v)
            (fn_x_sin(u) - fn_x_sin(v)).abs <= (Real.1 + two_pi) * (u - v).abs
        }
    }
    forall(t: Real) {
        if interval_contains(Real.0, two_pi, t) {
            fn_x_sin_abs_le_two_pi(t)
            fn_x_sin(t).abs <= two_pi
            abs_le_imp_lower(fn_x_sin(t), two_pi)
            -two_pi <= fn_x_sin(t)
        }
    }
    forall(t: Real) {
        if interval_contains(Real.0, two_pi, t) {
            fn_x_sin_abs_le_two_pi(t)
            fn_x_sin(t).abs <= two_pi
            abs_le_imp_upper(fn_x_sin(t), two_pi)
            fn_x_sin(t) <= two_pi
        }
    }
        Real.0 <= two_pi
    (Real.0 <= two_pi) and Real.0 <= Real.1 + two_pi
    ((Real.0 <= two_pi) and Real.0 <= Real.1 + two_pi) and continuous(g_x_sin)
    (((Real.0 <= two_pi) and Real.0 <= Real.1 + two_pi) and continuous(g_x_sin)) and is_derivative_fn(g_x_sin, fn_x_sin)
    ((((Real.0 <= two_pi) and Real.0 <= Real.1 + two_pi) and continuous(g_x_sin)) and is_derivative_fn(g_x_sin, fn_x_sin)) and forall(u: Real, v: Real) {
            interval_contains(Real.0, two_pi, u) and interval_contains(Real.0, two_pi, v)
            implies (fn_x_sin(u) - fn_x_sin(v)).abs <= (Real.1 + two_pi) * (u - v).abs
        }
    (((((Real.0 <= two_pi) and Real.0 <= Real.1 + two_pi) and continuous(g_x_sin)) and is_derivative_fn(g_x_sin, fn_x_sin)) and forall(u: Real, v: Real) {
            interval_contains(Real.0, two_pi, u) and interval_contains(Real.0, two_pi, v)
            implies (fn_x_sin(u) - fn_x_sin(v)).abs <= (Real.1 + two_pi) * (u - v).abs
        }) and forall(t: Real) { interval_contains(Real.0, two_pi, t) implies -two_pi <= fn_x_sin(t) }
    ((((((Real.0 <= two_pi) and Real.0 <= Real.1 + two_pi) and continuous(g_x_sin)) and is_derivative_fn(g_x_sin, fn_x_sin)) and forall(u: Real, v: Real) {
            interval_contains(Real.0, two_pi, u) and interval_contains(Real.0, two_pi, v)
            implies (fn_x_sin(u) - fn_x_sin(v)).abs <= (Real.1 + two_pi) * (u - v).abs
        }) and forall(t: Real) { interval_contains(Real.0, two_pi, t) implies -two_pi <= fn_x_sin(t) }) and forall(t: Real) { interval_contains(Real.0, two_pi, t) implies fn_x_sin(t) <= two_pi }
    if ((((((Real.0 <= two_pi) and Real.0 <= Real.1 + two_pi) and continuous(g_x_sin)) and is_derivative_fn(g_x_sin, fn_x_sin)) and forall(u: Real, v: Real) {
            interval_contains(Real.0, two_pi, u) and interval_contains(Real.0, two_pi, v)
            implies (fn_x_sin(u) - fn_x_sin(v)).abs <= (Real.1 + two_pi) * (u - v).abs
        }) and forall(t: Real) { interval_contains(Real.0, two_pi, t) implies -two_pi <= fn_x_sin(t) }) and forall(t: Real) { interval_contains(Real.0, two_pi, t) implies fn_x_sin(t) <= two_pi } {
        fn_integrable_gen(fn_x_sin, g_x_sin, Real.0, two_pi, Real.1 + two_pi, -two_pi, two_pi)
        is_integrable(fn_x_sin, Real.0, two_pi)
    }
}

// ---------------------------------------------------------------------------
// The integral computations (the concrete orthogonality results)
// ---------------------------------------------------------------------------

/// The integral of (x).sincos(x) over [0, 2pi] is zero (cross orthogonality,
/// m = n = 1).
theorem integral_sin_mul_cos_zero {
    integral(fn_sincos, Real.0, two_pi) = Real.0
} by {
    two_pi_nonneg
    Real.0 <= two_pi
    g_sincos_continuous
    continuous(g_sincos)
    g_sincos_has_derivative
    is_derivative_fn(g_sincos, fn_sincos)
    fn_sincos_integrable
    is_integrable(fn_sincos, Real.0, two_pi)
    forall(t: Real) {
        if interval_contains(Real.0, two_pi, t) {
            fn_sincos_abs_le_one(t)
            fn_sincos(t).abs <= Real.1
            abs_le_one_imp_lower(fn_sincos(t))
            -Real.1 <= fn_sincos(t)
        }
    }
    forall(t: Real) {
        if interval_contains(Real.0, two_pi, t) {
            fn_sincos_abs_le_one(t)
            fn_sincos(t).abs <= Real.1
            abs_le_one_imp_upper(fn_sincos(t))
            fn_sincos(t) <= Real.1
        }
    }
        Real.0 <= two_pi
    (Real.0 <= two_pi) and continuous(g_sincos)
    ((Real.0 <= two_pi) and continuous(g_sincos)) and is_derivative_fn(g_sincos, fn_sincos)
    (((Real.0 <= two_pi) and continuous(g_sincos)) and is_derivative_fn(g_sincos, fn_sincos)) and is_integrable(fn_sincos, Real.0, two_pi)
    ((((Real.0 <= two_pi) and continuous(g_sincos)) and is_derivative_fn(g_sincos, fn_sincos)) and is_integrable(fn_sincos, Real.0, two_pi)) and forall(t: Real) { interval_contains(Real.0, two_pi, t) implies -Real.1 <= fn_sincos(t) }
    (((((Real.0 <= two_pi) and continuous(g_sincos)) and is_derivative_fn(g_sincos, fn_sincos)) and is_integrable(fn_sincos, Real.0, two_pi)) and forall(t: Real) { interval_contains(Real.0, two_pi, t) implies -Real.1 <= fn_sincos(t) }) and forall(t: Real) { interval_contains(Real.0, two_pi, t) implies fn_sincos(t) <= Real.1 }
    if (((((Real.0 <= two_pi) and continuous(g_sincos)) and is_derivative_fn(g_sincos, fn_sincos)) and is_integrable(fn_sincos, Real.0, two_pi)) and forall(t: Real) { interval_contains(Real.0, two_pi, t) implies -Real.1 <= fn_sincos(t) }) and forall(t: Real) { interval_contains(Real.0, two_pi, t) implies fn_sincos(t) <= Real.1 } {
        ftc2_general(fn_sincos, g_sincos, Real.0, two_pi, -Real.1, Real.1)
        integral(fn_sincos, Real.0, two_pi) = g_sincos(two_pi) - g_sincos(Real.0)
        pointwise_mul_apply(constant[Real, Real](Real.1 / two), pointwise_mul(Real.sin, Real.sin), two_pi)
        pointwise_mul(constant[Real, Real](Real.1 / two), pointwise_mul(Real.sin, Real.sin), two_pi) =
            constant[Real, Real](Real.1 / two, two_pi) * pointwise_mul(Real.sin, Real.sin, two_pi)
        constant[Real, Real](Real.1 / two, two_pi) = Real.1 / two
        pointwise_mul_apply(Real.sin, Real.sin, two_pi)
        pointwise_mul(Real.sin, Real.sin, two_pi) = two_pi.sin * two_pi.sin
        g_sincos(two_pi) = (Real.1 / two) * (two_pi.sin * two_pi.sin)
        sin_two_pi_zero
        two_pi.sin = Real.0
        two_pi.sin * two_pi.sin = Real.0
        g_sincos(two_pi) = (Real.1 / two) * Real.0
        (Real.1 / two) * Real.0 = Real.0
        g_sincos(two_pi) = Real.0
        pointwise_mul_apply(constant[Real, Real](Real.1 / two), pointwise_mul(Real.sin, Real.sin), Real.0)
        pointwise_mul(constant[Real, Real](Real.1 / two), pointwise_mul(Real.sin, Real.sin), Real.0) =
            constant[Real, Real](Real.1 / two, Real.0) * pointwise_mul(Real.sin, Real.sin, Real.0)
        constant[Real, Real](Real.1 / two, Real.0) = Real.1 / two
        pointwise_mul_apply(Real.sin, Real.sin, Real.0)
        pointwise_mul(Real.sin, Real.sin, Real.0) = (Real.0).sin * (Real.0).sin
        g_sincos(Real.0) = (Real.1 / two) * ((Real.0).sin * (Real.0).sin)
        sin_zero
        (Real.0).sin = Real.0
        (Real.0).sin * (Real.0).sin = Real.0
        g_sincos(Real.0) = (Real.1 / two) * Real.0
        g_sincos(Real.0) = Real.0
        integral(fn_sincos, Real.0, two_pi) = Real.0 - Real.0
        Real.0 - Real.0 = Real.0
        integral(fn_sincos, Real.0, two_pi) = Real.0
    }
}
/// The integral of x.sin^2 over [0, 2pi] is pi (normalization, n = 1).
theorem integral_sin_sq_zero_two_pi {
    integral(fn_sin_sq, Real.0, two_pi) = pi
} by {
    two_pi_nonneg
    Real.0 <= two_pi
    g_sin_sq_continuous
    continuous(g_sin_sq)
    g_sin_sq_has_derivative
    is_derivative_fn(g_sin_sq, fn_sin_sq)
    fn_sin_sq_integrable
    is_integrable(fn_sin_sq, Real.0, two_pi)
    forall(t: Real) {
        if interval_contains(Real.0, two_pi, t) {
            fn_sin_sq_abs_le_one(t)
            fn_sin_sq(t).abs <= Real.1
            abs_le_one_imp_lower(fn_sin_sq(t))
            -Real.1 <= fn_sin_sq(t)
        }
    }
    forall(t: Real) {
        if interval_contains(Real.0, two_pi, t) {
            fn_sin_sq_abs_le_one(t)
            fn_sin_sq(t).abs <= Real.1
            abs_le_one_imp_upper(fn_sin_sq(t))
            fn_sin_sq(t) <= Real.1
        }
    }
        Real.0 <= two_pi
    (Real.0 <= two_pi) and continuous(g_sin_sq)
    ((Real.0 <= two_pi) and continuous(g_sin_sq)) and is_derivative_fn(g_sin_sq, fn_sin_sq)
    (((Real.0 <= two_pi) and continuous(g_sin_sq)) and is_derivative_fn(g_sin_sq, fn_sin_sq)) and is_integrable(fn_sin_sq, Real.0, two_pi)
    ((((Real.0 <= two_pi) and continuous(g_sin_sq)) and is_derivative_fn(g_sin_sq, fn_sin_sq)) and is_integrable(fn_sin_sq, Real.0, two_pi)) and forall(t: Real) { interval_contains(Real.0, two_pi, t) implies -Real.1 <= fn_sin_sq(t) }
    (((((Real.0 <= two_pi) and continuous(g_sin_sq)) and is_derivative_fn(g_sin_sq, fn_sin_sq)) and is_integrable(fn_sin_sq, Real.0, two_pi)) and forall(t: Real) { interval_contains(Real.0, two_pi, t) implies -Real.1 <= fn_sin_sq(t) }) and forall(t: Real) { interval_contains(Real.0, two_pi, t) implies fn_sin_sq(t) <= Real.1 }
    if (((((Real.0 <= two_pi) and continuous(g_sin_sq)) and is_derivative_fn(g_sin_sq, fn_sin_sq)) and is_integrable(fn_sin_sq, Real.0, two_pi)) and forall(t: Real) { interval_contains(Real.0, two_pi, t) implies -Real.1 <= fn_sin_sq(t) }) and forall(t: Real) { interval_contains(Real.0, two_pi, t) implies fn_sin_sq(t) <= Real.1 } {
        ftc2_general(fn_sin_sq, g_sin_sq, Real.0, two_pi, -Real.1, Real.1)
        integral(fn_sin_sq, Real.0, two_pi) = g_sin_sq(two_pi) - g_sin_sq(Real.0)
        pointwise_mul_apply(constant[Real, Real](Real.1 / two),
            pointwise_add(identity_fn[Real], pointwise_neg(pointwise_mul(Real.sin, Real.cos))), two_pi)
        pointwise_mul(constant[Real, Real](Real.1 / two),
            pointwise_add(identity_fn[Real], pointwise_neg(pointwise_mul(Real.sin, Real.cos))), two_pi) =
            constant[Real, Real](Real.1 / two, two_pi) *
                pointwise_add(identity_fn[Real], pointwise_neg(pointwise_mul(Real.sin, Real.cos)), two_pi)
        constant[Real, Real](Real.1 / two, two_pi) = Real.1 / two
        pointwise_add_apply(identity_fn[Real], pointwise_neg(pointwise_mul(Real.sin, Real.cos)), two_pi)
        pointwise_add(identity_fn[Real], pointwise_neg(pointwise_mul(Real.sin, Real.cos)), two_pi) =
            identity_fn[Real](two_pi) + pointwise_neg(pointwise_mul(Real.sin, Real.cos), two_pi)
        identity_fn[Real](two_pi) = two_pi
        pointwise_neg_apply(pointwise_mul(Real.sin, Real.cos), two_pi)
        pointwise_neg(pointwise_mul(Real.sin, Real.cos), two_pi) = -(pointwise_mul(Real.sin, Real.cos, two_pi))
        pointwise_mul_apply(Real.sin, Real.cos, two_pi)
        pointwise_mul(Real.sin, Real.cos, two_pi) = two_pi.sin * two_pi.cos
        pointwise_neg(pointwise_mul(Real.sin, Real.cos), two_pi) = -(two_pi.sin * two_pi.cos)
        g_sin_sq(two_pi) = (Real.1 / two) * (two_pi + -(two_pi.sin * two_pi.cos))
        sin_two_pi_zero
        two_pi.sin = Real.0
        cos_two_pi_one
        two_pi.cos = Real.1
        two_pi.sin * two_pi.cos = Real.0
        two_pi + -(two_pi.sin * two_pi.cos) = two_pi
        g_sin_sq(two_pi) = (Real.1 / two) * two_pi
        two_pi = pi + pi
        two * pi = pi + pi
        two_pi = two * pi
        half_twice(pi)
        (Real.1 / two) * (two * pi) = pi
        (Real.1 / two) * two_pi = pi
        g_sin_sq(two_pi) = pi
        pointwise_mul_apply(constant[Real, Real](Real.1 / two),
            pointwise_add(identity_fn[Real], pointwise_neg(pointwise_mul(Real.sin, Real.cos))), Real.0)
        pointwise_mul(constant[Real, Real](Real.1 / two),
            pointwise_add(identity_fn[Real], pointwise_neg(pointwise_mul(Real.sin, Real.cos))), Real.0) =
            constant[Real, Real](Real.1 / two, Real.0) *
                pointwise_add(identity_fn[Real], pointwise_neg(pointwise_mul(Real.sin, Real.cos)), Real.0)
        constant[Real, Real](Real.1 / two, Real.0) = Real.1 / two
        pointwise_add_apply(identity_fn[Real], pointwise_neg(pointwise_mul(Real.sin, Real.cos)), Real.0)
        pointwise_add(identity_fn[Real], pointwise_neg(pointwise_mul(Real.sin, Real.cos)), Real.0) =
            identity_fn[Real](Real.0) + pointwise_neg(pointwise_mul(Real.sin, Real.cos), Real.0)
        identity_fn[Real](Real.0) = Real.0
        pointwise_neg_apply(pointwise_mul(Real.sin, Real.cos), Real.0)
        pointwise_neg(pointwise_mul(Real.sin, Real.cos), Real.0) = -(pointwise_mul(Real.sin, Real.cos, Real.0))
        pointwise_mul_apply(Real.sin, Real.cos, Real.0)
        pointwise_mul(Real.sin, Real.cos, Real.0) = (Real.0).sin * (Real.0).cos
        pointwise_neg(pointwise_mul(Real.sin, Real.cos), Real.0) = -((Real.0).sin * (Real.0).cos)
        sin_zero
        (Real.0).sin = Real.0
        cos_zero
        (Real.0).cos = Real.1
        (Real.0).sin * (Real.0).cos = Real.0
        identity_fn[Real](Real.0) + pointwise_neg(pointwise_mul(Real.sin, Real.cos), Real.0) = Real.0
        g_sin_sq(Real.0) = (Real.1 / two) * Real.0
        (Real.1 / two) * Real.0 = Real.0
        g_sin_sq(Real.0) = Real.0
        integral(fn_sin_sq, Real.0, two_pi) = pi - Real.0
        pi - Real.0 = pi
        integral(fn_sin_sq, Real.0, two_pi) = pi
    }
}
/// The integral of x.cos^2 over [0, 2pi] is pi (normalization, n = 1).
theorem integral_cos_sq_zero_two_pi {
    integral(fn_cos_sq, Real.0, two_pi) = pi
} by {
    two_pi_nonneg
    Real.0 <= two_pi
    g_cos_sq_continuous
    continuous(g_cos_sq)
    g_cos_sq_has_derivative
    is_derivative_fn(g_cos_sq, fn_cos_sq)
    fn_cos_sq_integrable
    is_integrable(fn_cos_sq, Real.0, two_pi)
    forall(t: Real) {
        if interval_contains(Real.0, two_pi, t) {
            fn_cos_sq_abs_le_one(t)
            fn_cos_sq(t).abs <= Real.1
            abs_le_one_imp_lower(fn_cos_sq(t))
            -Real.1 <= fn_cos_sq(t)
        }
    }
    forall(t: Real) {
        if interval_contains(Real.0, two_pi, t) {
            fn_cos_sq_abs_le_one(t)
            fn_cos_sq(t).abs <= Real.1
            abs_le_one_imp_upper(fn_cos_sq(t))
            fn_cos_sq(t) <= Real.1
        }
    }
        Real.0 <= two_pi
    (Real.0 <= two_pi) and continuous(g_cos_sq)
    ((Real.0 <= two_pi) and continuous(g_cos_sq)) and is_derivative_fn(g_cos_sq, fn_cos_sq)
    (((Real.0 <= two_pi) and continuous(g_cos_sq)) and is_derivative_fn(g_cos_sq, fn_cos_sq)) and is_integrable(fn_cos_sq, Real.0, two_pi)
    ((((Real.0 <= two_pi) and continuous(g_cos_sq)) and is_derivative_fn(g_cos_sq, fn_cos_sq)) and is_integrable(fn_cos_sq, Real.0, two_pi)) and forall(t: Real) { interval_contains(Real.0, two_pi, t) implies -Real.1 <= fn_cos_sq(t) }
    (((((Real.0 <= two_pi) and continuous(g_cos_sq)) and is_derivative_fn(g_cos_sq, fn_cos_sq)) and is_integrable(fn_cos_sq, Real.0, two_pi)) and forall(t: Real) { interval_contains(Real.0, two_pi, t) implies -Real.1 <= fn_cos_sq(t) }) and forall(t: Real) { interval_contains(Real.0, two_pi, t) implies fn_cos_sq(t) <= Real.1 }
    if (((((Real.0 <= two_pi) and continuous(g_cos_sq)) and is_derivative_fn(g_cos_sq, fn_cos_sq)) and is_integrable(fn_cos_sq, Real.0, two_pi)) and forall(t: Real) { interval_contains(Real.0, two_pi, t) implies -Real.1 <= fn_cos_sq(t) }) and forall(t: Real) { interval_contains(Real.0, two_pi, t) implies fn_cos_sq(t) <= Real.1 } {
        ftc2_general(fn_cos_sq, g_cos_sq, Real.0, two_pi, -Real.1, Real.1)
        integral(fn_cos_sq, Real.0, two_pi) = g_cos_sq(two_pi) - g_cos_sq(Real.0)
        pointwise_mul_apply(constant[Real, Real](Real.1 / two),
            pointwise_add(identity_fn[Real], pointwise_mul(Real.sin, Real.cos)), two_pi)
        pointwise_mul(constant[Real, Real](Real.1 / two),
            pointwise_add(identity_fn[Real], pointwise_mul(Real.sin, Real.cos)), two_pi) =
            constant[Real, Real](Real.1 / two, two_pi) *
                pointwise_add(identity_fn[Real], pointwise_mul(Real.sin, Real.cos), two_pi)
        constant[Real, Real](Real.1 / two, two_pi) = Real.1 / two
        pointwise_add_apply(identity_fn[Real], pointwise_mul(Real.sin, Real.cos), two_pi)
        pointwise_add(identity_fn[Real], pointwise_mul(Real.sin, Real.cos), two_pi) =
            identity_fn[Real](two_pi) + pointwise_mul(Real.sin, Real.cos, two_pi)
        identity_fn[Real](two_pi) = two_pi
        pointwise_mul_apply(Real.sin, Real.cos, two_pi)
        pointwise_mul(Real.sin, Real.cos, two_pi) = two_pi.sin * two_pi.cos
        g_cos_sq(two_pi) = (Real.1 / two) * (two_pi + two_pi.sin * two_pi.cos)
        sin_two_pi_zero
        two_pi.sin = Real.0
        cos_two_pi_one
        two_pi.cos = Real.1
        two_pi.sin * two_pi.cos = Real.0
        two_pi + two_pi.sin * two_pi.cos = two_pi
        g_cos_sq(two_pi) = (Real.1 / two) * two_pi
        two_pi = pi + pi
        two * pi = pi + pi
        two_pi = two * pi
        half_twice(pi)
        (Real.1 / two) * (two * pi) = pi
        (Real.1 / two) * two_pi = pi
        g_cos_sq(two_pi) = pi
        pointwise_mul_apply(constant[Real, Real](Real.1 / two),
            pointwise_add(identity_fn[Real], pointwise_mul(Real.sin, Real.cos)), Real.0)
        pointwise_mul(constant[Real, Real](Real.1 / two),
            pointwise_add(identity_fn[Real], pointwise_mul(Real.sin, Real.cos)), Real.0) =
            constant[Real, Real](Real.1 / two, Real.0) *
                pointwise_add(identity_fn[Real], pointwise_mul(Real.sin, Real.cos), Real.0)
        constant[Real, Real](Real.1 / two, Real.0) = Real.1 / two
        pointwise_add_apply(identity_fn[Real], pointwise_mul(Real.sin, Real.cos), Real.0)
        pointwise_add(identity_fn[Real], pointwise_mul(Real.sin, Real.cos), Real.0) =
            identity_fn[Real](Real.0) + pointwise_mul(Real.sin, Real.cos, Real.0)
        identity_fn[Real](Real.0) = Real.0
        pointwise_mul_apply(Real.sin, Real.cos, Real.0)
        pointwise_mul(Real.sin, Real.cos, Real.0) = (Real.0).sin * (Real.0).cos
        sin_zero
        (Real.0).sin = Real.0
        cos_zero
        (Real.0).cos = Real.1
        (Real.0).sin * (Real.0).cos = Real.0
        identity_fn[Real](Real.0) + pointwise_mul(Real.sin, Real.cos, Real.0) = Real.0
        g_cos_sq(Real.0) = (Real.1 / two) * Real.0
        g_cos_sq(Real.0) = Real.0
        integral(fn_cos_sq, Real.0, two_pi) = pi - Real.0
        pi - Real.0 = pi
        integral(fn_cos_sq, Real.0, two_pi) = pi
    }
}
/// The integral of x.sin(2x).sin over [0, 2pi] is zero (orthogonality,
/// m = 1, n = 2).
theorem integral_sin_sin2_zero {
    integral(fn_sin_sin2, Real.0, two_pi) = Real.0
} by {
    two_pi_nonneg
    Real.0 <= two_pi
    g_sin_sin2_continuous
    continuous(g_sin_sin2)
    g_sin_sin2_has_derivative
    is_derivative_fn(g_sin_sin2, fn_sin_sin2)
    fn_sin_sin2_integrable
    is_integrable(fn_sin_sin2, Real.0, two_pi)
    forall(t: Real) {
        if interval_contains(Real.0, two_pi, t) {
            fn_sin_sin2_abs_le_one(t)
            fn_sin_sin2(t).abs <= Real.1
            abs_le_one_imp_lower(fn_sin_sin2(t))
            -Real.1 <= fn_sin_sin2(t)
        }
    }
    forall(t: Real) {
        if interval_contains(Real.0, two_pi, t) {
            fn_sin_sin2_abs_le_one(t)
            fn_sin_sin2(t).abs <= Real.1
            abs_le_one_imp_upper(fn_sin_sin2(t))
            fn_sin_sin2(t) <= Real.1
        }
    }
        Real.0 <= two_pi
    (Real.0 <= two_pi) and continuous(g_sin_sin2)
    ((Real.0 <= two_pi) and continuous(g_sin_sin2)) and is_derivative_fn(g_sin_sin2, fn_sin_sin2)
    (((Real.0 <= two_pi) and continuous(g_sin_sin2)) and is_derivative_fn(g_sin_sin2, fn_sin_sin2)) and is_integrable(fn_sin_sin2, Real.0, two_pi)
    ((((Real.0 <= two_pi) and continuous(g_sin_sin2)) and is_derivative_fn(g_sin_sin2, fn_sin_sin2)) and is_integrable(fn_sin_sin2, Real.0, two_pi)) and forall(t: Real) { interval_contains(Real.0, two_pi, t) implies -Real.1 <= fn_sin_sin2(t) }
    (((((Real.0 <= two_pi) and continuous(g_sin_sin2)) and is_derivative_fn(g_sin_sin2, fn_sin_sin2)) and is_integrable(fn_sin_sin2, Real.0, two_pi)) and forall(t: Real) { interval_contains(Real.0, two_pi, t) implies -Real.1 <= fn_sin_sin2(t) }) and forall(t: Real) { interval_contains(Real.0, two_pi, t) implies fn_sin_sin2(t) <= Real.1 }
    if (((((Real.0 <= two_pi) and continuous(g_sin_sin2)) and is_derivative_fn(g_sin_sin2, fn_sin_sin2)) and is_integrable(fn_sin_sin2, Real.0, two_pi)) and forall(t: Real) { interval_contains(Real.0, two_pi, t) implies -Real.1 <= fn_sin_sin2(t) }) and forall(t: Real) { interval_contains(Real.0, two_pi, t) implies fn_sin_sin2(t) <= Real.1 } {
        ftc2_general(fn_sin_sin2, g_sin_sin2, Real.0, two_pi, -Real.1, Real.1)
        integral(fn_sin_sin2, Real.0, two_pi) = g_sin_sin2(two_pi) - g_sin_sin2(Real.0)
        pointwise_mul_apply(constant[Real, Real](two / from_nat[Real](Nat.3)),
            pointwise_mul(pointwise_mul(Real.sin, Real.sin), Real.sin), two_pi)
        pointwise_mul(constant[Real, Real](two / from_nat[Real](Nat.3)),
            pointwise_mul(pointwise_mul(Real.sin, Real.sin), Real.sin), two_pi) =
            constant[Real, Real](two / from_nat[Real](Nat.3), two_pi) *
                pointwise_mul(pointwise_mul(Real.sin, Real.sin), Real.sin, two_pi)
        constant[Real, Real](two / from_nat[Real](Nat.3), two_pi) = two / from_nat[Real](Nat.3)
        pointwise_mul_apply(pointwise_mul(Real.sin, Real.sin), Real.sin, two_pi)
        pointwise_mul(pointwise_mul(Real.sin, Real.sin), Real.sin, two_pi) =
            pointwise_mul(Real.sin, Real.sin, two_pi) * two_pi.sin
        pointwise_mul_apply(Real.sin, Real.sin, two_pi)
        pointwise_mul(Real.sin, Real.sin, two_pi) = two_pi.sin * two_pi.sin
        g_sin_sin2(two_pi) =
            (two / from_nat[Real](Nat.3)) * (two_pi.sin * two_pi.sin * two_pi.sin)
        sin_two_pi_zero
        two_pi.sin = Real.0
        two_pi.sin * two_pi.sin * two_pi.sin = Real.0
        g_sin_sin2(two_pi) = (two / from_nat[Real](Nat.3)) * Real.0
        (two / from_nat[Real](Nat.3)) * Real.0 = Real.0
        g_sin_sin2(two_pi) = Real.0
        pointwise_mul_apply(constant[Real, Real](two / from_nat[Real](Nat.3)),
            pointwise_mul(pointwise_mul(Real.sin, Real.sin), Real.sin), Real.0)
        pointwise_mul(constant[Real, Real](two / from_nat[Real](Nat.3)),
            pointwise_mul(pointwise_mul(Real.sin, Real.sin), Real.sin), Real.0) =
            constant[Real, Real](two / from_nat[Real](Nat.3), Real.0) *
                pointwise_mul(pointwise_mul(Real.sin, Real.sin), Real.sin, Real.0)
        constant[Real, Real](two / from_nat[Real](Nat.3), Real.0) = two / from_nat[Real](Nat.3)
        pointwise_mul_apply(pointwise_mul(Real.sin, Real.sin), Real.sin, Real.0)
        pointwise_mul(pointwise_mul(Real.sin, Real.sin), Real.sin, Real.0) =
            pointwise_mul(Real.sin, Real.sin, Real.0) * (Real.0).sin
        pointwise_mul_apply(Real.sin, Real.sin, Real.0)
        pointwise_mul(Real.sin, Real.sin, Real.0) = (Real.0).sin * (Real.0).sin
        g_sin_sin2(Real.0) =
            (two / from_nat[Real](Nat.3)) * ((Real.0).sin * (Real.0).sin * (Real.0).sin)
        sin_zero
        (Real.0).sin = Real.0
        (Real.0).sin * (Real.0).sin * (Real.0).sin = Real.0
        g_sin_sin2(Real.0) = (two / from_nat[Real](Nat.3)) * Real.0
        g_sin_sin2(Real.0) = Real.0
        integral(fn_sin_sin2, Real.0, two_pi) = Real.0 - Real.0
        Real.0 - Real.0 = Real.0
        integral(fn_sin_sin2, Real.0, two_pi) = Real.0
    }
}
/// The integral of x x.sin over [0, 2pi] is -2pi (integration by parts).
theorem integral_x_sin_zero_two_pi {
    integral(fn_x_sin, Real.0, two_pi) = -two_pi
} by {
    two_pi_nonneg
    Real.0 <= two_pi
    g_x_sin_continuous
    continuous(g_x_sin)
    g_x_sin_has_derivative
    is_derivative_fn(g_x_sin, fn_x_sin)
    fn_x_sin_integrable
    is_integrable(fn_x_sin, Real.0, two_pi)
    forall(t: Real) {
        if interval_contains(Real.0, two_pi, t) {
            fn_x_sin_abs_le_two_pi(t)
            fn_x_sin(t).abs <= two_pi
            abs_le_imp_lower(fn_x_sin(t), two_pi)
            -two_pi <= fn_x_sin(t)
        }
    }
    forall(t: Real) {
        if interval_contains(Real.0, two_pi, t) {
            fn_x_sin_abs_le_two_pi(t)
            fn_x_sin(t).abs <= two_pi
            abs_le_imp_upper(fn_x_sin(t), two_pi)
            fn_x_sin(t) <= two_pi
        }
    }
        Real.0 <= two_pi
    (Real.0 <= two_pi) and continuous(g_x_sin)
    ((Real.0 <= two_pi) and continuous(g_x_sin)) and is_derivative_fn(g_x_sin, fn_x_sin)
    (((Real.0 <= two_pi) and continuous(g_x_sin)) and is_derivative_fn(g_x_sin, fn_x_sin)) and is_integrable(fn_x_sin, Real.0, two_pi)
    ((((Real.0 <= two_pi) and continuous(g_x_sin)) and is_derivative_fn(g_x_sin, fn_x_sin)) and is_integrable(fn_x_sin, Real.0, two_pi)) and forall(t: Real) { interval_contains(Real.0, two_pi, t) implies -two_pi <= fn_x_sin(t) }
    (((((Real.0 <= two_pi) and continuous(g_x_sin)) and is_derivative_fn(g_x_sin, fn_x_sin)) and is_integrable(fn_x_sin, Real.0, two_pi)) and forall(t: Real) { interval_contains(Real.0, two_pi, t) implies -two_pi <= fn_x_sin(t) }) and forall(t: Real) { interval_contains(Real.0, two_pi, t) implies fn_x_sin(t) <= two_pi }
    if (((((Real.0 <= two_pi) and continuous(g_x_sin)) and is_derivative_fn(g_x_sin, fn_x_sin)) and is_integrable(fn_x_sin, Real.0, two_pi)) and forall(t: Real) { interval_contains(Real.0, two_pi, t) implies -two_pi <= fn_x_sin(t) }) and forall(t: Real) { interval_contains(Real.0, two_pi, t) implies fn_x_sin(t) <= two_pi } {
        ftc2_general(fn_x_sin, g_x_sin, Real.0, two_pi, -two_pi, two_pi)
        integral(fn_x_sin, Real.0, two_pi) = g_x_sin(two_pi) - g_x_sin(Real.0)
        pointwise_add_apply(Real.sin, pointwise_neg(pointwise_mul(identity_fn[Real], Real.cos)), two_pi)
        pointwise_add(Real.sin, pointwise_neg(pointwise_mul(identity_fn[Real], Real.cos)), two_pi) =
            two_pi.sin + pointwise_neg(pointwise_mul(identity_fn[Real], Real.cos), two_pi)
        pointwise_neg_apply(pointwise_mul(identity_fn[Real], Real.cos), two_pi)
        pointwise_neg(pointwise_mul(identity_fn[Real], Real.cos), two_pi) =
            -(pointwise_mul(identity_fn[Real], Real.cos, two_pi))
        pointwise_mul_apply(identity_fn[Real], Real.cos, two_pi)
        pointwise_mul(identity_fn[Real], Real.cos, two_pi) = identity_fn[Real](two_pi) * two_pi.cos
        identity_fn[Real](two_pi) = two_pi
        pointwise_neg(pointwise_mul(identity_fn[Real], Real.cos), two_pi) = -(two_pi * two_pi.cos)
        g_x_sin(two_pi) = two_pi.sin + -(two_pi * two_pi.cos)
        sin_two_pi_zero
        two_pi.sin = Real.0
        cos_two_pi_one
        two_pi.cos = Real.1
        two_pi * two_pi.cos = two_pi
        g_x_sin(two_pi) = Real.0 + -two_pi
        Real.0 + -two_pi = -two_pi
        g_x_sin(two_pi) = -two_pi
        pointwise_add_apply(Real.sin, pointwise_neg(pointwise_mul(identity_fn[Real], Real.cos)), Real.0)
        pointwise_add(Real.sin, pointwise_neg(pointwise_mul(identity_fn[Real], Real.cos)), Real.0) =
            (Real.0).sin + pointwise_neg(pointwise_mul(identity_fn[Real], Real.cos), Real.0)
        pointwise_neg_apply(pointwise_mul(identity_fn[Real], Real.cos), Real.0)
        pointwise_neg(pointwise_mul(identity_fn[Real], Real.cos), Real.0) =
            -(pointwise_mul(identity_fn[Real], Real.cos, Real.0))
        pointwise_mul_apply(identity_fn[Real], Real.cos, Real.0)
        pointwise_mul(identity_fn[Real], Real.cos, Real.0) = identity_fn[Real](Real.0) * (Real.0).cos
        identity_fn[Real](Real.0) = Real.0
        pointwise_neg(pointwise_mul(identity_fn[Real], Real.cos), Real.0) = -(Real.0 * (Real.0).cos)
        Real.0 * (Real.0).cos = Real.0
        sin_zero
        (Real.0).sin = Real.0
        g_x_sin(Real.0) = Real.0 + -Real.0
        Real.0 + -Real.0 = Real.0
        g_x_sin(Real.0) = Real.0
        integral(fn_x_sin, Real.0, two_pi) = -two_pi - Real.0
        -two_pi - Real.0 = -two_pi
        integral(fn_x_sin, Real.0, two_pi) = -two_pi
    }
}

/// The first sine Fourier coefficient of f(x) = x on [0, 2pi] is -2:
/// b_1 = (1/pi) * integral(x x.sin, 0, 2pi) = -2.
theorem fourier_sin_coeff_identity_one {
    fourier_sin_coeff(identity_fn[Real], Nat.1) = -two
} by {
    forall(x: Real) {
        fourier_sin_integrand(identity_fn[Real], Nat.1, x) =
            identity_fn[Real](x) * sin_nt(Nat.1, x)
        identity_fn[Real](x) = x
        sin_nt(Nat.1, x) = (from_nat[Real](Nat.1) * x).sin
        from_nat[Real](Nat.1) = Real.1
        from_nat[Real](Nat.1) * x = x
        sin_nt(Nat.1, x) = x.sin
        fourier_sin_integrand(identity_fn[Real], Nat.1, x) = x * x.sin
        fn_x_sin(x) = identity_fn[Real](x) * x.sin
        fn_x_sin(x) = x * x.sin
        fourier_sin_integrand(identity_fn[Real], Nat.1, x) = fn_x_sin(x)
    }
    function_extensionality(fourier_sin_integrand(identity_fn[Real], Nat.1), fn_x_sin)
    fourier_sin_integrand(identity_fn[Real], Nat.1) = fn_x_sin
    integral(fourier_sin_integrand(identity_fn[Real], Nat.1), Real.0, two_pi) =
        integral(fn_x_sin, Real.0, two_pi)
    integral_x_sin_zero_two_pi
    integral(fn_x_sin, Real.0, two_pi) = -two_pi
    integral(fourier_sin_integrand(identity_fn[Real], Nat.1), Real.0, two_pi) = -two_pi
    fourier_sin_coeff(identity_fn[Real], Nat.1) =
        integral(fourier_sin_integrand(identity_fn[Real], Nat.1), Real.0, two_pi) / pi
    fourier_sin_coeff(identity_fn[Real], Nat.1) = -two_pi / pi
    pi_pos
    pi > Real.0
    lt_imp_ne(Real.0, pi)
    Real.0 != pi
    pi != Real.0
    two_pi = pi + pi
    two * pi = pi + pi
    two_pi = two * pi
    div_mul_cancel_left(pi, two)
    pi != Real.0 implies (pi * two) / pi = two
    (pi * two) / pi = two
    (two * pi) / pi = two
    -(two * pi) / pi = -two
    -two_pi / pi = -two
    fourier_sin_coeff(identity_fn[Real], Nat.1) = -two
}

// ---------------------------------------------------------------------------
// The general statements (stated, not proved here)
// ---------------------------------------------------------------------------
//
// The general orthogonality and normalization relations for arbitrary natural
// numbers m and n, and Parseval's identity, are stated below as comments.
// Each needs the integral of nx.sin or nx.cos for general n, which requires
// a linear-substitution rule for the Darboux integral; that rule is not yet in
// the library.  The concrete instances proved above (m, n in {1, 2}) are the
// foundations on which these general statements will rest.
//
// /// The trigonometric system is orthogonal on [0, 2pi]: for m != n,
// /// integral(mx.sin(nx).sin, 0, 2pi) = 0.
// // theorem integral_sin_mx_sin_nx_zero(m: Nat, n: Nat) {
// //     m != n
// //     implies
// //     integral(fourier_sin_integrand(sin_nt(m), n), Real.0, two_pi) = Real.0
// // }
//
// /// The sine functions are normalized on [0, 2pi]:
// /// integral(nx.sin^2, 0, 2pi) = pi for every n >= 1.
// // theorem integral_sin_nx_sq_pi(n: Nat) {
// //     n != Nat.0 implies integral(fn_sin_sq, Real.0, two_pi) = pi
// // }
//
// /// The cross terms vanish: integral((mx).sincos(nx), 0, 2pi) = 0 for all
// /// m and n.
// // theorem integral_sin_mx_cos_nx_zero(m: Nat, n: Nat) {
// //     integral(fourier_cos_integrand(sin_nt(m), n), Real.0, two_pi) = Real.0
// // }
//
// /// Parseval's identity: the mean square of f equals the sum of the squares
// /// of its Fourier coefficients (with the a_0 term halved):
// /// (1/pi) integral(f^2, 0, 2pi) =
// ///     a_0^2 / 2 + sum_{n=1}^infinity (a_n^2 + b_n^2).
// // theorem parseval_identity(f: Real -> Real) {
// //     ...
// // }
