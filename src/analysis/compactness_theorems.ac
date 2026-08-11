/// Compactness theorems for closed bounded intervals of the real line.
///
/// This file collects the classical compactness theorems of real analysis
/// for closed bounded intervals of the real line, restating the library's
/// results:
///
///   - the intermediate value theorem: a continuous function on `[a, b]`
///     takes every value between its endpoint values.  Restated below with
///     a proof from `intermediate_value_closed_interval`, the library's
///     bisection proof (`real/interface.ac`);
///   - the Heine-Borel theorem: the closed interval `[a, b]` is compact in
///     the open-cover sense.  Restated below with a proof from
///     `heine_borel_closed_interval`, the open-cover compactness proof
///     (`analysis/topology/real_topology.ac`);
///   - the extreme value theorem: a continuous function on `[a, b]` attains
///     a maximum on the interval.  Proved inside the library as
///     `continuous_closed_interval_attains_maximum` in
///     `real/mean_value.ac`, but that module is private to the `real`
///     package, so the statement is recorded below without proof;
///   - uniform continuity on `[a, b]` (Heine-Cantor): a function continuous
///     on `[a, b]` is uniformly continuous there.  Recorded without proof:
///     every classical argument needs the Bolzano-Weierstrass theorem, a
///     nested-interval construction, or a Lebesgue-number argument, none of
///     which is formalized yet (see the discussion in
///     `real/uniform_continuity.ac`); and
///   - the Bolzano-Weierstrass theorem: every bounded sequence of real
///     numbers has a convergent subsequence.  Recorded without proof: the
///     monotone-subsequence (peaks) construction is not yet in the library
///     (see `theorems1000/theorem_bolzano_weierstrass.ac`).

from nat import Nat
from real import Real, continuous, continuous_at, intermediate_value_closed_interval
from order_set import closed_interval_set, closed_interval_set_lower_le,
    closed_interval_set_le_upper
from analysis.topology.topological_space import is_compact
from analysis.topology.real_topology import heine_borel_closed_interval

/// True if f is continuous at every point of the closed interval [lower, upper].
define continuous_on_closed(f: Real -> Real, lower: Real, upper: Real) -> Bool {
    forall(x: Real) {
        closed_interval_set(lower, upper).contains(x) implies continuous_at(f, x)
    }
}

/// True if f satisfies the delta-epsilon condition for uniform continuity on
/// the closed interval [lower, upper].
define uniform_on_condition(f: Real -> Real, lower: Real, upper: Real, delta: Real, eps: Real) -> Bool {
    forall(x: Real, y: Real) {
        closed_interval_set(lower, upper).contains(x) and
        closed_interval_set(lower, upper).contains(y) and
        x.is_close(y, delta)
        implies f(x).is_close(f(y), eps)
    }
}

/// True if f is uniformly continuous on the closed interval [lower, upper].
define uniform_on(f: Real -> Real, lower: Real, upper: Real) -> Bool {
    forall(eps: Real) {
        eps.is_positive implies exists(delta: Real) {
            delta.is_positive and uniform_on_condition(f, lower, upper, delta, eps)
        }
    }
}

/// True if a natural-indexed map is unbounded, i.e. not bounded above by any
/// fixed value.
define is_unbounded(f: Nat -> Nat) -> Bool {
    forall(bound: Nat) {
        exists(n: Nat) {
            bound < f(n)
        }
    }
}

/// The intermediate value theorem: a continuous real function on a closed
/// bounded interval takes every value between its endpoint values.
theorem intermediate_value_on_closed_interval(f: Real -> Real, a: Real, b: Real, t: Real) {
    continuous(f) and a <= b and f(a) <= t and t <= f(b)
    implies exists(c: Real) { a <= c and c <= b and f(c) = t }
} by {
    if continuous(f) and a <= b and f(a) <= t and t <= f(b) {
        intermediate_value_closed_interval(f, a, b, t)
        exists(point: Real) {
            closed_interval_set(a, b).contains(point) and f(point) = t
        }
        let point: Real satisfy {
            closed_interval_set(a, b).contains(point) and f(point) = t
        }
        closed_interval_set_lower_le(a, b, point)
        a <= point
        closed_interval_set_le_upper(a, b, point)
        point <= b
        exists(c: Real) { a <= c and c <= b and f(c) = t }
    }
}

/// The Heine-Borel theorem: a closed bounded interval of the real line is
/// compact, i.e. every open cover of it admits a finite subcover.
theorem heine_borel_closed_interval_is_compact(a: Real, b: Real) {
    is_compact[Real](closed_interval_set(a, b))
} by {
    heine_borel_closed_interval(a, b)
    is_compact[Real](closed_interval_set(a, b))
}

// The extreme value theorem: a continuous function on a closed bounded
// interval attains a maximum on the interval.  The library proves this as
// `continuous_closed_interval_attains_maximum` in `real/mean_value.ac`
// (take the supremum of the image, then use boundedness of the reciprocal
// of `m - f` to rule out a non-attained supremum); that module is private
// to the `real` package, so the statement is recorded here without proof.
//
// theorem extreme_value_on_closed_interval(f: Real -> Real, a: Real, b: Real) {
//     continuous_on_closed(f, a, b) and a < b
//     implies exists(x0: Real) {
//         closed_interval_set(a, b).contains(x0) and
//         (forall(y: Real) { closed_interval_set(a, b).contains(y) implies f(y) <= f(x0) })
//     }
// }

// The Heine-Cantor theorem: a function continuous on a closed bounded
// interval is uniformly continuous on it.  Every classical proof needs one
// of the Bolzano-Weierstrass theorem, a nested-interval (bisection)
// construction, or a Lebesgue-number argument, and none of the three is
// formalized yet; see the extended discussion in
// `real/uniform_continuity.ac`.  The predicate `continuous_on_closed`
// defined above is the library's `continuous_on` of
// `real/uniform_continuity.ac`.
//
// theorem heine_cantor_closed_interval(f: Real -> Real, lower: Real, upper: Real) {
//     continuous_on_closed(f, lower, upper) and lower <= upper
//     implies uniform_on(f, lower, upper)
// }

// The Bolzano-Weierstrass theorem: every bounded sequence of real numbers
// has a convergent subsequence, expressed through a monotone unbounded
// index map `f` and a limit `l` (using the library's `subsequence(a, f)`
// and `converges_to`).  The proof needs the peaks construction of a
// monotone subsequence of an arbitrary sequence together with the monotone
// convergence principle, and the construction is not yet in the library;
// see `theorems1000/theorem_bolzano_weierstrass.ac`.
//
// theorem bolzano_weierstrass(a: Nat -> Real) {
//     exists(lo: Real, hi: Real) {
//         forall(n: Nat) { lo <= a(n) and a(n) <= hi }
//     } implies
//         exists(f: Nat -> Nat, l: Real) {
//             is_monotone(f) and is_unbounded(f) and
//             converges_to(subsequence(a, f), l)
//         }
// }
