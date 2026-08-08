from nat import Nat
from data.basic.set import Set
from analysis.dynamical.dynamical_systems import iterate, orbit_contains, orbit,
    is_invariant, orbit_subset_invariant,
    is_fixed_point, fixed_point_set, is_periodic, is_periodic_point,
    has_period_dividing, periodic_point_set, period_dividing_set,
    is_prime_period, prime_period_set, is_periodic_point_iterate,
    is_periodic_iterate, has_period_dividing_iff_periodic_point,
    is_semiconjugacy, is_conjugacy, orbit_semiconjugacy,
    fixed_point_semiconjugacy, periodic_point_semiconjugacy,
    periodic_semiconjugacy, conjugacy_fixed_point_iff
from analysis.dynamical.dynamical_systems_periodic import period_dividing_set_invariant
from analysis.dynamical.dynamical_conjugacy_periods import conjugacy_has_period_dividing_iff,
    conjugacy_periodic_iff, conjugacy_prime_period_iff

/// Membership in a forward orbit unfolds to the explicit existential predicate.
theorem orbit_contains_iff[T](f: T -> T, x: T, y: T) {
    orbit(f, x).contains(y) = orbit_contains(f, x, y)
} by {
}

/// Membership in the prime-period set unfolds to the prime-period predicate.
theorem prime_period_set_contains_eq[T](f: T -> T, n: Nat, x: T) {
    prime_period_set(f, n).contains(x) = is_prime_period(f, x, n)
} by {
    prime_period_set(f, n) = Set[T].new(function(y: T) { is_prime_period(f, y, n) })
    x ∈ Set[T].new(function(y: T) { is_prime_period(f, y, n) }) =
        function(y: T) { is_prime_period(f, y, n) }(x)
    function(y: T) { is_prime_period(f, y, n) }(x) = is_prime_period(f, x, n)
}

/// Fixed-point set membership is exactly the fixed-point predicate.
theorem fixed_point_set_membership_iff[T](f: T -> T, x: T) {
    fixed_point_set(f).contains(x) = is_fixed_point(f, x)
} by {
}

/// Periodic-point set membership is exactly existential periodicity.
theorem periodic_point_set_membership_iff[T](f: T -> T, x: T) {
    periodic_point_set(f).contains(x) = is_periodic(f, x)
} by {
}

/// Period-dividing set membership is exactly the period-dividing predicate.
theorem period_dividing_set_membership_iff[T](f: T -> T, n: Nat, x: T) {
    period_dividing_set(f, n).contains(x) = has_period_dividing(f, x, n)
} by {
    period_dividing_set(f, n) = Set[T].new(function(y: T) { has_period_dividing(f, y, n) })
    x ∈ Set[T].new(function(y: T) { has_period_dividing(f, y, n) }) =
        function(y: T) { has_period_dividing(f, y, n) }(x)
    function(y: T) { has_period_dividing(f, y, n) }(x) = has_period_dividing(f, x, n)
}

/// Prime-period set membership is exactly the prime-period predicate.
theorem prime_period_set_membership_iff[T](f: T -> T, n: Nat, x: T) {
    prime_period_set(f, n).contains(x) = is_prime_period(f, x, n)
} by {
    prime_period_set_contains_eq(f, n, x)
}

/// A point in an invariant set carries its entire forward orbit into that set.
theorem invariant_orbit_member[T](f: T -> T, s: Set[T], x: T, y: T) {
    is_invariant(f, s) and s.contains(x) and orbit(f, x).contains(y) implies s.contains(y)
} by {
    if is_invariant(f, s) and s.contains(x) and orbit(f, x).contains(y) {
        orbit_subset_invariant(f, s, x)
        orbit(f, x).subset(s)
        s.contains(y)
    }
}

/// Every point in the orbit of a period-n point also has period n.
theorem orbit_member_periodic_point_same_period[T](f: T -> T, x: T, y: T, n: Nat) {
    is_periodic_point(f, x, n) and orbit(f, x).contains(y) implies is_periodic_point(f, y, n)
} by {
    if is_periodic_point(f, x, n) and orbit(f, x).contains(y) {
        orbit_contains_iff(f, x, y)
        let k: Nat satisfy {
            y = iterate(f, k, x)
        }
        is_periodic_point_iterate(f, x, n, k)
        is_periodic_point(f, y, n)
    }
}

/// Every point in the orbit of a point with period dividing n also has period dividing n.
theorem orbit_member_period_dividing_same_period[T](f: T -> T, x: T, y: T, n: Nat) {
    has_period_dividing(f, x, n) and orbit(f, x).contains(y) implies has_period_dividing(f, y, n)
} by {
    if has_period_dividing(f, x, n) and orbit(f, x).contains(y) {
        has_period_dividing_iff_periodic_point(f, x, n)
        orbit_member_periodic_point_same_period(f, x, y, n)
        has_period_dividing_iff_periodic_point(f, y, n)
        has_period_dividing(f, y, n)
    }
}

/// A period-dividing point has its whole orbit in the corresponding period-dividing set.
theorem period_dividing_orbit_subset[T](f: T -> T, x: T, n: Nat) {
    period_dividing_set(f, n).contains(x) implies orbit(f, x).subset(period_dividing_set(f, n))
} by {
    if period_dividing_set(f, n).contains(x) {
        period_dividing_set_invariant(f, n)
        orbit_subset_invariant(f, period_dividing_set(f, n), x)
    }
}

/// A periodic point has every forward iterate periodic.
theorem periodic_orbit_iterate_periodic[T](f: T -> T, x: T, k: Nat) {
    periodic_point_set(f).contains(x) implies periodic_point_set(f).contains(iterate(f, k, x))
} by {
    if periodic_point_set(f).contains(x) {
        periodic_point_set_membership_iff(f, x)
        is_periodic_iterate(f, x, k)
        periodic_point_set_membership_iff(f, iterate(f, k, x))
        periodic_point_set(f).contains(iterate(f, k, x))
    }
}

/// A semiconjugacy maps source orbit membership to target orbit membership.
theorem semiconjugacy_orbit_member[T, U](h: T -> U, f: T -> T, g: U -> U, x: T, y: T) {
    is_semiconjugacy(h, f, g) and orbit(f, x).contains(y) implies orbit(g, h(x)).contains(h(y))
} by {
    if is_semiconjugacy(h, f, g) and orbit(f, x).contains(y) {
        orbit_semiconjugacy(h, f, g, x, y)
    }
}

/// A semiconjugacy maps fixed-set members to fixed-set members.
theorem semiconjugacy_fixed_point_set_member[T, U](h: T -> U, f: T -> T, g: U -> U, x: T) {
    is_semiconjugacy(h, f, g) and fixed_point_set(f).contains(x) implies
    fixed_point_set(g).contains(h(x))
} by {
    if is_semiconjugacy(h, f, g) and fixed_point_set(f).contains(x) {
        fixed_point_set_membership_iff(f, x)
        fixed_point_semiconjugacy(h, f, g, x)
        fixed_point_set_membership_iff(g, h(x))
        fixed_point_set(g).contains(h(x))
    }
}

/// A semiconjugacy maps period-dividing-set members to period-dividing-set members.
theorem semiconjugacy_period_dividing_set_member[T, U](h: T -> U, f: T -> T, g: U -> U, x: T, n: Nat) {
    is_semiconjugacy(h, f, g) and period_dividing_set(f, n).contains(x) implies
    period_dividing_set(g, n).contains(h(x))
} by {
    if is_semiconjugacy(h, f, g) and period_dividing_set(f, n).contains(x) {
        period_dividing_set_membership_iff(f, n, x)
        has_period_dividing_iff_periodic_point(f, x, n)
        periodic_point_semiconjugacy(h, f, g, x, n)
        has_period_dividing_iff_periodic_point(g, h(x), n)
        period_dividing_set_membership_iff(g, n, h(x))
        period_dividing_set(g, n).contains(h(x))
    }
}

/// A semiconjugacy maps periodic-set members to periodic-set members.
theorem semiconjugacy_periodic_point_set_member[T, U](h: T -> U, f: T -> T, g: U -> U, x: T) {
    is_semiconjugacy(h, f, g) and periodic_point_set(f).contains(x) implies
    periodic_point_set(g).contains(h(x))
} by {
    if is_semiconjugacy(h, f, g) and periodic_point_set(f).contains(x) {
        periodic_point_set_membership_iff(f, x)
        periodic_semiconjugacy(h, f, g, x)
        periodic_point_set_membership_iff(g, h(x))
        periodic_point_set(g).contains(h(x))
    }
}

/// Conjugacy gives an iff for fixed-point-set membership.
theorem conjugacy_fixed_point_set_member_iff[T, U](h: T -> U, k: U -> T, f: T -> T, g: U -> U, x: T) {
    is_conjugacy(h, k, f, g) implies
    fixed_point_set(f).contains(x) = fixed_point_set(g).contains(h(x))
} by {
    if is_conjugacy(h, k, f, g) {
        fixed_point_set_membership_iff(f, x)
        fixed_point_set_membership_iff(g, h(x))
        conjugacy_fixed_point_iff(h, k, f, g, x)
    }
}

/// Conjugacy gives an iff for period-dividing-set membership.
theorem conjugacy_period_dividing_set_member_iff[T, U](h: T -> U, k: U -> T, f: T -> T, g: U -> U, x: T, n: Nat) {
    is_conjugacy(h, k, f, g) implies
    period_dividing_set(f, n).contains(x) = period_dividing_set(g, n).contains(h(x))
} by {
    if is_conjugacy(h, k, f, g) {
        period_dividing_set_membership_iff(f, n, x)
        period_dividing_set_membership_iff(g, n, h(x))
        conjugacy_has_period_dividing_iff(h, k, f, g, x, n)
    }
}

/// Conjugacy gives an iff for periodic-point-set membership.
theorem conjugacy_periodic_point_set_member_iff[T, U](h: T -> U, k: U -> T, f: T -> T, g: U -> U, x: T) {
    is_conjugacy(h, k, f, g) implies
    periodic_point_set(f).contains(x) = periodic_point_set(g).contains(h(x))
} by {
    if is_conjugacy(h, k, f, g) {
        periodic_point_set_membership_iff(f, x)
        periodic_point_set_membership_iff(g, h(x))
        conjugacy_periodic_iff(h, k, f, g, x)
    }
}

/// Conjugacy gives an iff for prime-period-set membership.
theorem conjugacy_prime_period_set_member_iff[T, U](h: T -> U, k: U -> T, f: T -> T, g: U -> U, x: T, n: Nat) {
    is_conjugacy(h, k, f, g) implies
    prime_period_set(f, n).contains(x) = prime_period_set(g, n).contains(h(x))
} by {
    if is_conjugacy(h, k, f, g) {
        prime_period_set_contains_eq(f, n, x)
        prime_period_set_contains_eq(g, n, h(x))
        conjugacy_prime_period_iff(h, k, f, g, x, n)
    }
}
