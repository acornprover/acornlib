from nat import Nat
from data.basic.set import Set, subset_trans, subset_contains_eq, set_image,
    set_image_contains_witness
from analysis.dynamical.dynamical_systems import is_invariant, orbit, fixed_point_set,
    periodic_point_set, period_dividing_set, prime_period_set,
    is_fixed_point, is_periodic, is_periodic_point, has_period_dividing,
    is_fixed_point_is_periodic, has_period_dividing_iff_periodic_point,
    is_periodic_point_step, prime_period_set_subset_period_dividing_set,
    period_dividing_set_subset_periodic, periodic_point_set_invariant,
    orbit_subset_invariant
from analysis.dynamical.dynamical_systems import is_semiconjugacy, orbit_semiconjugacy,
    fixed_point_semiconjugacy, periodic_point_semiconjugacy,
    periodic_semiconjugacy

/// Membership in the fixed-point set unfolds to the fixed-point predicate.
lemma fixed_point_set_contains_eq[T](f: T -> T, x: T) {
    fixed_point_set(f).contains(x) = is_fixed_point(f, x)
}

/// Membership in the periodic-point set unfolds to periodicity.
lemma periodic_point_set_contains_eq[T](f: T -> T, x: T) {
    periodic_point_set(f).contains(x) = is_periodic(f, x)
}

/// Membership in the period-dividing set unfolds to `has_period_dividing`.
lemma period_dividing_set_contains_eq[T](f: T -> T, n: Nat, x: T) {
    period_dividing_set(f, n).contains(x) = has_period_dividing(f, x, n)
} by {
    period_dividing_set(f, n) = Set[T].new(function(y: T) { has_period_dividing(f, y, n) })
    x ∈ Set[T].new(function(y: T) { has_period_dividing(f, y, n) }) =
        function(y: T) { has_period_dividing(f, y, n) }(x)
    function(y: T) { has_period_dividing(f, y, n) }(x) = has_period_dividing(f, x, n)
}

/// Fixed points are periodic points, with period one.
theorem fixed_point_set_subset_periodic_point_set[T](f: T -> T) {
    fixed_point_set(f).subset(periodic_point_set(f))
} by {
    if not fixed_point_set(f).subset(periodic_point_set(f)) {
        subset_contains_eq(fixed_point_set(f), periodic_point_set(f))
        let x: T satisfy {
            fixed_point_set(f).contains(x) and not periodic_point_set(f).contains(x)
        }
        fixed_point_set_contains_eq(f, x)
        is_fixed_point_is_periodic(f, x)
        periodic_point_set_contains_eq(f, x)
        false
    }
}

/// The set of points returning after `n` iterates is forward-invariant.
theorem period_dividing_set_invariant[T](f: T -> T, n: Nat) {
    is_invariant(f, period_dividing_set(f, n))
} by {
    if not is_invariant(f, period_dividing_set(f, n)) {
        let x: T satisfy {
            period_dividing_set(f, n).contains(x) and not period_dividing_set(f, n).contains(f(x))
        }
        period_dividing_set_contains_eq(f, n, x)
        has_period_dividing_iff_periodic_point(f, x, n)
        is_periodic_point_step(f, x, n)
        has_period_dividing(f, f(x), n)
        period_dividing_set_contains_eq(f, n, f(x))
        false
    }
}

/// Prime-period points are periodic points.
theorem prime_period_set_subset_periodic_point_set[T](f: T -> T, n: Nat) {
    prime_period_set(f, n).subset(periodic_point_set(f))
} by {
    prime_period_set_subset_period_dividing_set(f, n)
    period_dividing_set_subset_periodic(f, n)
    subset_trans(prime_period_set(f, n), period_dividing_set(f, n), periodic_point_set(f))
}

/// The forward orbit of a periodic point stays inside the periodic-point set.
theorem periodic_orbit_subset_periodic_point_set[T](f: T -> T, x: T) {
    is_periodic(f, x) implies orbit(f, x).subset(periodic_point_set(f))
} by {
    if is_periodic(f, x) {
        periodic_point_set_invariant(f)
        orbit_subset_invariant(f, periodic_point_set(f), x)
        orbit(f, x).subset(periodic_point_set(f))
    }
}

/// A semiconjugacy maps the forward orbit of a point into the forward orbit of its image.
theorem orbit_semiconjugacy_image_subset[T, U](h: T -> U, f: T -> T, g: U -> U, x: T) {
    is_semiconjugacy(h, f, g) implies set_image(orbit(f, x), h).subset(orbit(g, h(x)))
} by {
    if is_semiconjugacy(h, f, g) {
        forall(y: U) {
            if set_image(orbit(f, x), h).contains(y) {
                set_image_contains_witness(orbit(f, x), h, y)
                let z: T satisfy {
                    orbit(f, x).contains(z) and y = h(z)
                }
                orbit_semiconjugacy(h, f, g, x, z)
                orbit(g, h(x)).contains(y)
            }
        }
    }
}

/// A semiconjugacy maps fixed points into fixed points.
theorem fixed_point_set_semiconjugacy_image_subset[T, U](h: T -> U, f: T -> T, g: U -> U) {
    is_semiconjugacy(h, f, g) implies set_image(fixed_point_set(f), h).subset(fixed_point_set(g))
} by {
    if is_semiconjugacy(h, f, g) {
        forall(y: U) {
            if set_image(fixed_point_set(f), h).contains(y) {
                set_image_contains_witness(fixed_point_set(f), h, y)
                let x: T satisfy {
                    fixed_point_set(f).contains(x) and y = h(x)
                }
                fixed_point_set_contains_eq(f, x)
                fixed_point_semiconjugacy(h, f, g, x)
                fixed_point_set_contains_eq(g, h(x))
                fixed_point_set(g).contains(h(x))
                fixed_point_set(g).contains(y)
            }
        }
    }
}

/// A semiconjugacy maps points with period dividing `n` into points with period dividing `n`.
theorem period_dividing_set_semiconjugacy_image_subset[T, U](h: T -> U, f: T -> T, g: U -> U, n: Nat) {
    is_semiconjugacy(h, f, g) implies
        set_image(period_dividing_set(f, n), h).subset(period_dividing_set(g, n))
} by {
    if is_semiconjugacy(h, f, g) {
        forall(y: U) {
            if set_image(period_dividing_set(f, n), h).contains(y) {
                set_image_contains_witness(period_dividing_set(f, n), h, y)
                let x: T satisfy {
                    period_dividing_set(f, n).contains(x) and y = h(x)
                }
                period_dividing_set_contains_eq(f, n, x)
                has_period_dividing_iff_periodic_point(f, x, n)
                periodic_point_semiconjugacy(h, f, g, x, n)
                has_period_dividing_iff_periodic_point(g, h(x), n)
                period_dividing_set_contains_eq(g, n, h(x))
                period_dividing_set(g, n).contains(h(x))
                period_dividing_set(g, n).contains(y)
            }
        }
    }
}

/// A semiconjugacy maps periodic points into periodic points.
theorem periodic_point_set_semiconjugacy_image_subset[T, U](h: T -> U, f: T -> T, g: U -> U) {
    is_semiconjugacy(h, f, g) implies
        set_image(periodic_point_set(f), h).subset(periodic_point_set(g))
} by {
    if is_semiconjugacy(h, f, g) {
        forall(y: U) {
            if set_image(periodic_point_set(f), h).contains(y) {
                set_image_contains_witness(periodic_point_set(f), h, y)
                let x: T satisfy {
                    periodic_point_set(f).contains(x) and y = h(x)
                }
                periodic_point_set_contains_eq(f, x)
                periodic_semiconjugacy(h, f, g, x)
                periodic_point_set_contains_eq(g, h(x))
                periodic_point_set(g).contains(h(x))
                periodic_point_set(g).contains(y)
            }
        }
    }
}
