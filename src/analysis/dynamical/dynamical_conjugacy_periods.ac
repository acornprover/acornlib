/// Conjugacy invariance of periods and forward orbits for discrete dynamical systems.

from nat import Nat
from data.basic.set import Set, set_ext, set_image, set_image_contains_witness, maps_into_set_image
from analysis.dynamical.dynamical_systems import iterate, orbit, fixed_point_set, periodic_point_set,
    period_dividing_set, prime_period_set,
    is_fixed_point, is_periodic_point, is_periodic,
    is_prime_period, has_period_dividing,
    is_semiconjugacy, is_conjugacy, iterate_semiconjugacy,
    orbit_semiconjugacy,
    conjugacy_is_semiconjugacy, conjugacy_reverse_semiconjugacy,
    conjugacy_fixed_point_iff,
    conjugacy_orbit_reverse, conjugacy_orbit_forward_inverse_image

/// Local membership unfolding for period-dividing sets.
lemma conjugacy_period_dividing_set_contains_eq[T](f: T -> T, n: Nat, x: T) {
    period_dividing_set(f, n).contains(x) = has_period_dividing(f, x, n)
} by {
    period_dividing_set(f, n) = Set[T].new(function(y: T) { has_period_dividing(f, y, n) })
    x ∈ Set[T].new(function(y: T) { has_period_dividing(f, y, n) }) =
        function(y: T) { has_period_dividing(f, y, n) }(x)
    function(y: T) { has_period_dividing(f, y, n) }(x) = has_period_dividing(f, x, n)
}

/// Local membership unfolding for prime-period sets.
lemma conjugacy_prime_period_set_contains_eq[T](f: T -> T, n: Nat, x: T) {
    prime_period_set(f, n).contains(x) = is_prime_period(f, x, n)
} by {
    prime_period_set(f, n) = Set[T].new(function(y: T) { is_prime_period(f, y, n) })
    x ∈ Set[T].new(function(y: T) { is_prime_period(f, y, n) }) =
        function(y: T) { is_prime_period(f, y, n) }(x)
    function(y: T) { is_prime_period(f, y, n) }(x) = is_prime_period(f, x, n)
}

/// Points fixed by the nth iterate of a self-map.
define iterate_fixed_point_set[T](f: T -> T, n: Nat) -> Set[T] {
    fixed_point_set(function(x: T) { iterate(f, n, x) })
}

/// Local membership unfolding for fixed points of an iterate.
lemma conjugacy_iterate_fixed_point_set_contains_eq[T](f: T -> T, n: Nat, x: T) {
    iterate_fixed_point_set(f, n).contains(x) = (iterate(f, n, x) = x)
} by {
    fixed_point_set(function(y: T) { iterate(f, n, y) }).contains(x) =
        is_fixed_point(function(y: T) { iterate(f, n, y) }, x)
    is_fixed_point(function(y: T) { iterate(f, n, y) }, x) = (iterate(f, n, x) = x)
}

/// A conjugacy carries a return time for x under f to the same return time for h(x) under g.
theorem conjugacy_iterate_self_forward[T, U](
    h: T -> U,
    k: U -> T,
    f: T -> T,
    g: U -> U,
    x: T,
    n: Nat
) {
    is_conjugacy(h, k, f, g) and iterate(f, n, x) = x
        implies iterate(g, n, h(x)) = h(x)
} by {
    if is_conjugacy(h, k, f, g) and iterate(f, n, x) = x {
        conjugacy_is_semiconjugacy(h, k, f, g)
        iterate_semiconjugacy(h, f, g, x, n)
        iterate(g, n, h(x)) = h(x)
    }
}

/// A conjugacy reflects a return time for h(x) under g back to x under f.
theorem conjugacy_iterate_self_backward[T, U](
    h: T -> U,
    k: U -> T,
    f: T -> T,
    g: U -> U,
    x: T,
    n: Nat
) {
    is_conjugacy(h, k, f, g) and iterate(g, n, h(x)) = h(x)
        implies iterate(f, n, x) = x
} by {
    if is_conjugacy(h, k, f, g) and iterate(g, n, h(x)) = h(x) {
        conjugacy_reverse_semiconjugacy(h, k, f, g)
        iterate_semiconjugacy(k, g, f, h(x), n)
        k(iterate(g, n, h(x))) = iterate(f, n, k(h(x)))
        is_conjugacy(h, k, f, g) =
            (is_semiconjugacy(h, f, g)
            and forall(z: T) { k(h(z)) = z }
            and forall(w: U) { h(k(w)) = w })
        forall(z: T) { k(h(z)) = z }
        iterate(f, n, x) = x
    }
}

/// Period-dividing membership is invariant under conjugacy.
theorem conjugacy_has_period_dividing_iff[T, U](
    h: T -> U,
    k: U -> T,
    f: T -> T,
    g: U -> U,
    x: T,
    n: Nat
) {
    is_conjugacy(h, k, f, g) implies
        has_period_dividing(f, x, n) = has_period_dividing(g, h(x), n)
} by {
    if is_conjugacy(h, k, f, g) {
        if has_period_dividing(f, x, n) {
            conjugacy_iterate_self_forward(h, k, f, g, x, n)
            iterate(g, n, h(x)) = h(x)
            has_period_dividing(g, h(x), n)
        }
        if has_period_dividing(g, h(x), n) {
            conjugacy_iterate_self_backward(h, k, f, g, x, n)
            iterate(f, n, x) = x
            has_period_dividing(f, x, n)
        }
        (has_period_dividing(f, x, n) = has_period_dividing(g, h(x), n)) = true
    }
}

/// Same-period periodic-point membership is invariant under conjugacy.
theorem conjugacy_periodic_point_iff[T, U](
    h: T -> U,
    k: U -> T,
    f: T -> T,
    g: U -> U,
    x: T,
    n: Nat
) {
    is_conjugacy(h, k, f, g) implies
        is_periodic_point(f, x, n) = is_periodic_point(g, h(x), n)
} by {
    if is_conjugacy(h, k, f, g) {
        if is_periodic_point(f, x, n) {
            conjugacy_iterate_self_forward(h, k, f, g, x, n)
            iterate(g, n, h(x)) = h(x)
            is_periodic_point(g, h(x), n)
        }
        if is_periodic_point(g, h(x), n) {
            conjugacy_iterate_self_backward(h, k, f, g, x, n)
            iterate(f, n, x) = x
            is_periodic_point(f, x, n)
        }
        (is_periodic_point(f, x, n) = is_periodic_point(g, h(x), n)) = true
    }
}

/// Existential periodicity is invariant under conjugacy.
theorem conjugacy_periodic_iff[T, U](
    h: T -> U,
    k: U -> T,
    f: T -> T,
    g: U -> U,
    x: T
) {
    is_conjugacy(h, k, f, g) implies is_periodic(f, x) = is_periodic(g, h(x))
} by {
    if is_conjugacy(h, k, f, g) {
        if is_periodic(f, x) {
            let n: Nat satisfy {
                is_periodic_point(f, x, n)
            }
            conjugacy_periodic_point_iff(h, k, f, g, x, n)
            is_periodic(g, h(x))
        }
        if is_periodic(g, h(x)) {
            let n: Nat satisfy {
                is_periodic_point(g, h(x), n)
            }
            conjugacy_periodic_point_iff(h, k, f, g, x, n)
            is_periodic(f, x)
        }
        (is_periodic(f, x) = is_periodic(g, h(x))) = true
    }
}

/// Universal instantiation for natural-number predicates.
theorem forall_nat_apply(p: Nat -> Bool, m: Nat) {
    (forall(q: Nat) { p(q) }) implies p(m)
} by {
    if forall(q: Nat) { p(q) } {
        p(m)
    }
}

/// A quantified smaller-period exclusion applies at a smaller positive index.
theorem prime_period_minimal_from_forall[T](f: T -> T, x: T, n: Nat, m: Nat) {
    (forall(q: Nat) { q < n and q != Nat.0 implies iterate(f, q, x) != x })
        and m < n and m != Nat.0 implies iterate(f, m, x) != x
} by {
    if (forall(q: Nat) { q < n and q != Nat.0 implies iterate(f, q, x) != x })
        and m < n and m != Nat.0 {
        forall_nat_apply(function(q: Nat) {
            q < n and q != Nat.0 implies iterate(f, q, x) != x
        }, m)
        if iterate(f, m, x) = x {
            false
        }
        iterate(f, m, x) != x
    }
}

/// Exact (prime) period is invariant under conjugacy.
theorem conjugacy_prime_period_iff[T, U](
    h: T -> U,
    k: U -> T,
    f: T -> T,
    g: U -> U,
    x: T,
    n: Nat
) {
    is_conjugacy(h, k, f, g) implies
        is_prime_period(f, x, n) = is_prime_period(g, h(x), n)
} by {
    if is_conjugacy(h, k, f, g) {
        if is_prime_period(f, x, n) {
            is_prime_period(f, x, n) =
                (is_periodic_point(f, x, n)
                and forall(m: Nat) {
                    m < n and m != Nat.0 implies iterate(f, m, x) != x
                })
            conjugacy_periodic_point_iff(h, k, f, g, x, n)
            forall(m: Nat) {
                if m < n and m != Nat.0 {
                    if iterate(g, m, h(x)) = h(x) {
                        conjugacy_iterate_self_backward(h, k, f, g, x, m)
                        prime_period_minimal_from_forall(f, x, n, m)
                        false
                    }
                    iterate(g, m, h(x)) != h(x)
                }
            }
            is_prime_period(g, h(x), n)
        }
        if is_prime_period(g, h(x), n) {
            is_prime_period(g, h(x), n) =
                (is_periodic_point(g, h(x), n)
                and forall(m: Nat) {
                    m < n and m != Nat.0 implies iterate(g, m, h(x)) != h(x)
                })
            conjugacy_periodic_point_iff(h, k, f, g, x, n)
            forall(m: Nat) {
                if m < n and m != Nat.0 {
                    if iterate(f, m, x) = x {
                        conjugacy_iterate_self_forward(h, k, f, g, x, m)
                        prime_period_minimal_from_forall(g, h(x), n, m)
                        false
                    }
                    iterate(f, m, x) != x
                }
            }
            is_prime_period(f, x, n)
        }
        (is_prime_period(f, x, n) = is_prime_period(g, h(x), n)) = true
    }
}

/// A conjugacy identifies the image of a source orbit with the target orbit.
theorem conjugacy_orbit_image_eq[T, U](
    h: T -> U,
    k: U -> T,
    f: T -> T,
    g: U -> U,
    x: T
) {
    is_conjugacy(h, k, f, g) implies set_image(orbit(f, x), h) = orbit(g, h(x))
} by {
    if is_conjugacy(h, k, f, g) {
        forall(y: U) {
            if set_image(orbit(f, x), h).contains(y) {
                set_image_contains_witness(orbit(f, x), h, y)
                let z: T satisfy {
                    orbit(f, x).contains(z) and y = h(z)
                }
                conjugacy_is_semiconjugacy(h, k, f, g)
                orbit_semiconjugacy(h, f, g, x, z)
                orbit(g, h(x)).contains(y)
            }
            if orbit(g, h(x)).contains(y) {
                conjugacy_orbit_reverse(h, k, f, g, x, y)
                maps_into_set_image(orbit(f, x), h, k(y))
                set_image(orbit(f, x), h).contains(h(k(y)))
                is_conjugacy(h, k, f, g) =
                    (is_semiconjugacy(h, f, g)
                    and forall(z: T) { k(h(z)) = z }
                    and forall(w: U) { h(k(w)) = w })
                forall(w: U) { h(k(w)) = w }
                set_image(orbit(f, x), h).contains(y)
            }
            set_image(orbit(f, x), h).contains(y) = orbit(g, h(x)).contains(y)
        }
        set_ext(set_image(orbit(f, x), h), orbit(g, h(x)))
        set_image(orbit(f, x), h) = orbit(g, h(x))
    }
}

/// A conjugacy identifies the image of source fixed points with target fixed points.
theorem conjugacy_fixed_point_set_image_eq[T, U](
    h: T -> U,
    k: U -> T,
    f: T -> T,
    g: U -> U
) {
    is_conjugacy(h, k, f, g) implies set_image(fixed_point_set(f), h) = fixed_point_set(g)
} by {
    if is_conjugacy(h, k, f, g) {
        forall(y: U) {
            if set_image(fixed_point_set(f), h).contains(y) {
                set_image_contains_witness(fixed_point_set(f), h, y)
                let x: T satisfy {
                    fixed_point_set(f).contains(x) and y = h(x)
                }
                conjugacy_fixed_point_iff(h, k, f, g, x)
                fixed_point_set(g).contains(y)
            }
            if fixed_point_set(g).contains(y) {
                is_conjugacy(h, k, f, g) =
                    (is_semiconjugacy(h, f, g)
                    and forall(z: T) { k(h(z)) = z }
                    and forall(w: U) { h(k(w)) = w })
                forall(w: U) { h(k(w)) = w }
                conjugacy_fixed_point_iff(h, k, f, g, k(y))
                maps_into_set_image(fixed_point_set(f), h, k(y))
                set_image(fixed_point_set(f), h).contains(h(k(y)))
                set_image(fixed_point_set(f), h).contains(y)
            }
            set_image(fixed_point_set(f), h).contains(y) = fixed_point_set(g).contains(y)
        }
        set_ext(set_image(fixed_point_set(f), h), fixed_point_set(g))
        set_image(fixed_point_set(f), h) = fixed_point_set(g)
    }
}

/// A conjugacy identifies fixed points of the nth source iterate with fixed points of the nth target iterate.
theorem conjugacy_iterate_fixed_point_set_image_eq[T, U](
    h: T -> U,
    k: U -> T,
    f: T -> T,
    g: U -> U,
    n: Nat
) {
    is_conjugacy(h, k, f, g) implies
        set_image(iterate_fixed_point_set(f, n), h) = iterate_fixed_point_set(g, n)
} by {
    if is_conjugacy(h, k, f, g) {
        forall(y: U) {
            if set_image(iterate_fixed_point_set(f, n), h).contains(y) {
                set_image_contains_witness(iterate_fixed_point_set(f, n), h, y)
                let x: T satisfy {
                    iterate_fixed_point_set(f, n).contains(x) and y = h(x)
                }
                conjugacy_iterate_fixed_point_set_contains_eq(f, n, x)
                conjugacy_iterate_self_forward(h, k, f, g, x, n)
                iterate(g, n, h(x)) = h(x)
                conjugacy_iterate_fixed_point_set_contains_eq(g, n, h(x))
                iterate_fixed_point_set(g, n).contains(y)
            }
            if iterate_fixed_point_set(g, n).contains(y) {
                conjugacy_iterate_fixed_point_set_contains_eq(g, n, y)
                is_conjugacy(h, k, f, g) =
                    (is_semiconjugacy(h, f, g)
                    and forall(z: T) { k(h(z)) = z }
                    and forall(w: U) { h(k(w)) = w })
                forall(w: U) { h(k(w)) = w }
                conjugacy_iterate_self_backward(h, k, f, g, k(y), n)
                conjugacy_iterate_fixed_point_set_contains_eq(f, n, k(y))
                iterate_fixed_point_set(f, n).contains(k(y))
                maps_into_set_image(iterate_fixed_point_set(f, n), h, k(y))
                set_image(iterate_fixed_point_set(f, n), h).contains(y)
            }
            set_image(iterate_fixed_point_set(f, n), h).contains(y) =
                iterate_fixed_point_set(g, n).contains(y)
        }
        set_ext(set_image(iterate_fixed_point_set(f, n), h),
            iterate_fixed_point_set(g, n))
        set_image(iterate_fixed_point_set(f, n), h) =
            iterate_fixed_point_set(g, n)
    }
}

/// A conjugacy identifies the image of source periodic points with target periodic points.
theorem conjugacy_periodic_point_set_image_eq[T, U](
    h: T -> U,
    k: U -> T,
    f: T -> T,
    g: U -> U
) {
    is_conjugacy(h, k, f, g) implies set_image(periodic_point_set(f), h) = periodic_point_set(g)
} by {
    if is_conjugacy(h, k, f, g) {
        forall(y: U) {
            if set_image(periodic_point_set(f), h).contains(y) {
                set_image_contains_witness(periodic_point_set(f), h, y)
                let x: T satisfy {
                    periodic_point_set(f).contains(x) and y = h(x)
                }
                conjugacy_periodic_iff(h, k, f, g, x)
                periodic_point_set(g).contains(y)
            }
            if periodic_point_set(g).contains(y) {
                is_conjugacy(h, k, f, g) =
                    (is_semiconjugacy(h, f, g)
                    and forall(z: T) { k(h(z)) = z }
                    and forall(w: U) { h(k(w)) = w })
                forall(w: U) { h(k(w)) = w }
                conjugacy_periodic_iff(h, k, f, g, k(y))
                maps_into_set_image(periodic_point_set(f), h, k(y))
                set_image(periodic_point_set(f), h).contains(h(k(y)))
                set_image(periodic_point_set(f), h).contains(y)
            }
            set_image(periodic_point_set(f), h).contains(y) = periodic_point_set(g).contains(y)
        }
        set_ext(set_image(periodic_point_set(f), h), periodic_point_set(g))
        set_image(periodic_point_set(f), h) = periodic_point_set(g)
    }
}

/// A conjugacy identifies the image of source period-dividing points with target period-dividing points.
theorem conjugacy_period_dividing_set_image_eq[T, U](
    h: T -> U,
    k: U -> T,
    f: T -> T,
    g: U -> U,
    n: Nat
) {
    is_conjugacy(h, k, f, g) implies set_image(period_dividing_set(f, n), h) = period_dividing_set(g, n)
} by {
    if is_conjugacy(h, k, f, g) {
        forall(y: U) {
            if set_image(period_dividing_set(f, n), h).contains(y) {
                set_image_contains_witness(period_dividing_set(f, n), h, y)
                let x: T satisfy {
                    period_dividing_set(f, n).contains(x) and y = h(x)
                }
                conjugacy_period_dividing_set_contains_eq(f, n, x)
                conjugacy_has_period_dividing_iff(h, k, f, g, x, n)
                conjugacy_period_dividing_set_contains_eq(g, n, h(x))
                period_dividing_set(g, n).contains(y)
            }
            if period_dividing_set(g, n).contains(y) {
                conjugacy_period_dividing_set_contains_eq(g, n, y)
                is_conjugacy(h, k, f, g) =
                    (is_semiconjugacy(h, f, g)
                    and forall(z: T) { k(h(z)) = z }
                    and forall(w: U) { h(k(w)) = w })
                forall(w: U) { h(k(w)) = w }
                conjugacy_has_period_dividing_iff(h, k, f, g, k(y), n)
                conjugacy_period_dividing_set_contains_eq(f, n, k(y))
                maps_into_set_image(period_dividing_set(f, n), h, k(y))
                set_image(period_dividing_set(f, n), h).contains(h(k(y)))
                set_image(period_dividing_set(f, n), h).contains(y)
            }
            set_image(period_dividing_set(f, n), h).contains(y) = period_dividing_set(g, n).contains(y)
        }
        set_ext(set_image(period_dividing_set(f, n), h), period_dividing_set(g, n))
        set_image(period_dividing_set(f, n), h) = period_dividing_set(g, n)
    }
}

/// A conjugacy identifies the image of source prime-period points with target prime-period points.
theorem conjugacy_prime_period_set_image_eq[T, U](
    h: T -> U,
    k: U -> T,
    f: T -> T,
    g: U -> U,
    n: Nat
) {
    is_conjugacy(h, k, f, g) implies set_image(prime_period_set(f, n), h) = prime_period_set(g, n)
} by {
    if is_conjugacy(h, k, f, g) {
        forall(y: U) {
            if set_image(prime_period_set(f, n), h).contains(y) {
                set_image_contains_witness(prime_period_set(f, n), h, y)
                let x: T satisfy {
                    prime_period_set(f, n).contains(x) and y = h(x)
                }
                conjugacy_prime_period_set_contains_eq(f, n, x)
                conjugacy_prime_period_iff(h, k, f, g, x, n)
                conjugacy_prime_period_set_contains_eq(g, n, h(x))
                prime_period_set(g, n).contains(y)
            }
            if prime_period_set(g, n).contains(y) {
                conjugacy_prime_period_set_contains_eq(g, n, y)
                is_conjugacy(h, k, f, g) =
                    (is_semiconjugacy(h, f, g)
                    and forall(z: T) { k(h(z)) = z }
                    and forall(w: U) { h(k(w)) = w })
                forall(w: U) { h(k(w)) = w }
                conjugacy_prime_period_iff(h, k, f, g, k(y), n)
                conjugacy_prime_period_set_contains_eq(f, n, k(y))
                maps_into_set_image(prime_period_set(f, n), h, k(y))
                set_image(prime_period_set(f, n), h).contains(h(k(y)))
                set_image(prime_period_set(f, n), h).contains(y)
            }
            set_image(prime_period_set(f, n), h).contains(y) = prime_period_set(g, n).contains(y)
        }
        set_ext(set_image(prime_period_set(f, n), h), prime_period_set(g, n))
        set_image(prime_period_set(f, n), h) = prime_period_set(g, n)
    }
}

/// A target-side orbit point is in the transported orbit exactly when its inverse witness lies in the source orbit.
theorem conjugacy_orbit_contains_inverse_eq[T, U](
    h: T -> U,
    k: U -> T,
    f: T -> T,
    g: U -> U,
    x: T,
    y: U
) {
    is_conjugacy(h, k, f, g) implies
        orbit(g, h(x)).contains(y) = orbit(f, x).contains(k(y))
} by {
    if is_conjugacy(h, k, f, g) {
        if orbit(g, h(x)).contains(y) {
            conjugacy_orbit_reverse(h, k, f, g, x, y)
            orbit(f, x).contains(k(y))
        }
        if orbit(f, x).contains(k(y)) {
            conjugacy_orbit_forward_inverse_image(h, k, f, g, x, y)
            orbit(g, h(x)).contains(y)
        }
        (orbit(g, h(x)).contains(y) = orbit(f, x).contains(k(y))) = true
    }
}
