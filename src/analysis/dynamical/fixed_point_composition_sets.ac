/// Set-level fixed-point composition inclusions.
///
/// The base `fixed_point` module provides pointwise facts saying that common
/// fixed, pre-fixed, and post-fixed points remain fixed/pre-fixed/post-fixed for
/// a composition. This module packages those facts as direct inclusions between
/// the corresponding fixed-point sets.

from order import PartialOrder, is_monotone
from analysis.dynamical.fixed_point import is_fixed_point, is_pre_fixed_point, is_post_fixed_point,
    fixed_point_set, common_fixed_point_set,
    common_fixed_point_set_contains_eq, fixed_point_set_contains_eq,
    fixed_point_of_compose_of_common, pre_fixed_point_set,
    common_pre_fixed_point_set, common_pre_fixed_point_set_contains_eq,
    pre_fixed_point_set_contains_eq, pre_fixed_point_of_compose,
    post_fixed_point_set, common_post_fixed_point_set,
    common_post_fixed_point_set_contains_eq, post_fixed_point_set_contains_eq,
    post_fixed_point_of_compose
from data.basic.functions import compose
from data.basic.set import subset_contains_eq

/// Common fixed points of two self-maps are fixed by their composition.
theorem common_fixed_point_set_subset_compose[T](f: T -> T, g: T -> T) {
    common_fixed_point_set(f, g).subset(fixed_point_set(compose(f, g)))
} by {
    let target = fixed_point_set(compose(f, g))
    subset_contains_eq(common_fixed_point_set(f, g), target)
    forall(x: T) {
        if common_fixed_point_set(f, g).contains(x) {
            common_fixed_point_set_contains_eq(f, g, x)
            is_fixed_point(g, x)
            fixed_point_of_compose_of_common(f, g, x)
            is_fixed_point(compose(f, g), x)
            fixed_point_set_contains_eq(compose(f, g), x)
            target.contains(x)
        }
    }
    (forall(x: T) { not common_fixed_point_set(f, g).contains(x) or target.contains(x) }) = true
    common_fixed_point_set(f, g).subset(target)
}

/// Common pre-fixed points of two self-maps are pre-fixed by their composition
/// when the outer map is monotone.
theorem common_pre_fixed_point_set_subset_compose[P: PartialOrder](f: P -> P, g: P -> P) {
    is_monotone(f) implies
        common_pre_fixed_point_set(f, g).subset(pre_fixed_point_set(compose(f, g)))
} by {
    if is_monotone(f) {
        forall(x: P) {
            if common_pre_fixed_point_set(f, g).contains(x) {
                common_pre_fixed_point_set_contains_eq(f, g, x)
                is_pre_fixed_point(f, x)
                is_pre_fixed_point(g, x)
                pre_fixed_point_of_compose(f, g, x)
                is_pre_fixed_point(compose(f, g), x)
                pre_fixed_point_set_contains_eq(compose(f, g), x)
                pre_fixed_point_set(compose(f, g)).contains(x)
            }
        }
    }
}

/// Common post-fixed points of two self-maps are post-fixed by their composition
/// when the outer map is monotone.
theorem common_post_fixed_point_set_subset_compose[P: PartialOrder](f: P -> P, g: P -> P) {
    is_monotone(f) implies
        common_post_fixed_point_set(f, g).subset(post_fixed_point_set(compose(f, g)))
} by {
    if is_monotone(f) {
        forall(x: P) {
            if common_post_fixed_point_set(f, g).contains(x) {
                common_post_fixed_point_set_contains_eq(f, g, x)
                is_post_fixed_point(f, x)
                is_post_fixed_point(g, x)
                post_fixed_point_of_compose(f, g, x)
                is_post_fixed_point(compose(f, g), x)
                post_fixed_point_set_contains_eq(compose(f, g), x)
                post_fixed_point_set(compose(f, g)).contains(x)
            }
        }
    }
}
