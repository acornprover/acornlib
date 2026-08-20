/// Pre-fixed, post-fixed, and fixed points of self-maps.

from order import PartialOrder, lte_refl, lte_antisymm, lte_trans
from order import is_monotone, monotone_step
from analysis.dynamical.dynamical_systems import iterate, iterate_zero, iterate_suc, iterate_add,
    is_fixed_point, fixed_point_set
from nat import Nat
from data.basic.set import Set, set_ext, subset_contains, subset_contains_eq, subset_antisymm, subset_trans, elem_in_intersection, sets_subset_intersection
from data.basic.functions import compose, identity_fn
from algebra.field.galois_connection import is_fixed_by, is_closure_operator, is_kernel_operator,
    closure_operator_extensive_at, closure_operator_idempotent_at,
    kernel_operator_reductive_at, kernel_operator_idempotent_at,
    galois_closure, galois_kernel, is_galois_connection, galois_closure_extensive,
    galois_kernel_reductive, galois_closure_is_closure_operator, galois_kernel_is_kernel_operator

/// True if `x` is a pre-fixed point of `f`, i.e. `f(x) <= x`.
define is_pre_fixed_point[P: PartialOrder](f: P -> P, x: P) -> Bool {
    f(x) <= x
}

/// True if `x` is a post-fixed point of `f`, i.e. `x <= f(x)`.
define is_post_fixed_point[P: PartialOrder](f: P -> P, x: P) -> Bool {
    x <= f(x)
}

/// The set of pre-fixed points of an order-preserving self-map candidate.
define pre_fixed_point_set[P: PartialOrder](f: P -> P) -> Set[P] {
    Set[P].new(is_pre_fixed_point(f))
}

/// The set of post-fixed points of an order-preserving self-map candidate.
define post_fixed_point_set[P: PartialOrder](f: P -> P) -> Set[P] {
    Set[P].new(is_post_fixed_point(f))
}

/// Membership in the fixed-point set is being fixed by the map.
theorem fixed_point_set_contains_eq[T](f: T -> T, x: T) {
    fixed_point_set(f).contains(x) = is_fixed_point(f, x)
} by {
}

/// Membership in the pre-fixed-point set is being pre-fixed by the map.
theorem pre_fixed_point_set_contains_eq[P: PartialOrder](f: P -> P, x: P) {
    pre_fixed_point_set(f).contains(x) = is_pre_fixed_point(f, x)
} by {
}

/// Membership in the post-fixed-point set is being post-fixed by the map.
theorem post_fixed_point_set_contains_eq[P: PartialOrder](f: P -> P, x: P) {
    post_fixed_point_set(f).contains(x) = is_post_fixed_point(f, x)
} by {
}

/// Being a fixed point is the same as being fixed by the map.
theorem is_fixed_point_iff_is_fixed_by[T](f: T -> T, x: T) {
    is_fixed_point(f, x) = is_fixed_by(f, x)
} by {
    is_fixed_point(f, x) = (f(x) = x)
    is_fixed_by(f, x) = (f(x) = x)
}

/// A point fixed by a map is a fixed point of the map.
theorem fixed_point_of_fixed_by[T](f: T -> T, x: T) {
    is_fixed_by(f, x) implies is_fixed_point(f, x)
} by {
    if is_fixed_by(f, x) {
        is_fixed_by(f, x) = (f(x) = x)
        is_fixed_point(f, x)
    }
}

/// A fixed point of a map is fixed by the map.
theorem fixed_by_of_fixed_point[T](f: T -> T, x: T) {
    is_fixed_point(f, x) implies is_fixed_by(f, x)
} by {
    if is_fixed_point(f, x) {
        is_fixed_point(f, x) = (f(x) = x)
        is_fixed_by(f, x)
    }
}

/// Membership in the fixed-point set is being fixed by the map.
theorem fixed_point_set_contains_eq_fixed_by[T](f: T -> T, x: T) {
    fixed_point_set(f).contains(x) = is_fixed_by(f, x)
} by {
    fixed_point_set_contains_eq(f, x)
    is_fixed_point_iff_is_fixed_by(f, x)
    fixed_point_set(f).contains(x) = is_fixed_by(f, x)
}

/// A fixed point is a pre-fixed point.
theorem fixed_point_is_pre_fixed[P: PartialOrder](f: P -> P, x: P) {
    is_fixed_point(f, x) implies is_pre_fixed_point(f, x)
}

/// A fixed point is a post-fixed point.
theorem fixed_point_is_post_fixed[P: PartialOrder](f: P -> P, x: P) {
    is_fixed_point(f, x) implies is_post_fixed_point(f, x)
}

/// A point that is both pre-fixed and post-fixed is a fixed point.
theorem fixed_point_of_pre_and_post[P: PartialOrder](f: P -> P, x: P) {
    is_pre_fixed_point(f, x) and is_post_fixed_point(f, x) implies is_fixed_point(f, x)
} by {
    if is_pre_fixed_point(f, x) and is_post_fixed_point(f, x) {
        f(x) <= x
        x <= f(x)
        lte_antisymm(f(x), x)
    }
}

/// Fixed point as the conjunction of pre-fixed and post-fixed.
theorem fixed_point_iff_pre_and_post[P: PartialOrder](f: P -> P, x: P) {
    is_fixed_point(f, x) = (is_pre_fixed_point(f, x) and is_post_fixed_point(f, x))
}

/// Fixed points are pre-fixed points.
theorem fixed_point_set_subset_pre_fixed_point_set[P: PartialOrder](f: P -> P) {
    fixed_point_set(f).subset(pre_fixed_point_set(f))
} by {
    forall(x: P) {
        if fixed_point_set(f).contains(x) {
            fixed_point_set_contains_eq(f, x)
            is_fixed_point(f, x)
            fixed_point_is_pre_fixed(f, x)
            is_pre_fixed_point(f, x)
            pre_fixed_point_set_contains_eq(f, x)
            pre_fixed_point_set(f).contains(x)
        }
    }
}

/// Fixed points are post-fixed points.
theorem fixed_point_set_subset_post_fixed_point_set[P: PartialOrder](f: P -> P) {
    fixed_point_set(f).subset(post_fixed_point_set(f))
} by {
    forall(x: P) {
        if fixed_point_set(f).contains(x) {
            fixed_point_set_contains_eq(f, x)
            is_fixed_point(f, x)
            fixed_point_is_post_fixed(f, x)
            is_post_fixed_point(f, x)
            post_fixed_point_set_contains_eq(f, x)
            post_fixed_point_set(f).contains(x)
        }
    }
}

/// A point in both the pre-fixed and post-fixed sets is fixed.
theorem pre_post_fixed_point_sets_subset_fixed_point_set[P: PartialOrder](f: P -> P) {
    pre_fixed_point_set(f).intersection(post_fixed_point_set(f)).subset(fixed_point_set(f))
} by {
    forall(x: P) {
        if pre_fixed_point_set(f).intersection(post_fixed_point_set(f)).contains(x) {
            pre_fixed_point_set(f).contains(x)
            post_fixed_point_set(f).contains(x)
            pre_fixed_point_set_contains_eq(f, x)
            post_fixed_point_set_contains_eq(f, x)
            is_pre_fixed_point(f, x)
            is_post_fixed_point(f, x)
            fixed_point_of_pre_and_post(f, x)
            is_fixed_point(f, x)
            fixed_point_set_contains_eq(f, x)
            fixed_point_set(f).contains(x)
        }
    }
}

/// The fixed-point set is the intersection of the pre-fixed and post-fixed sets.
theorem fixed_point_set_eq_pre_post_intersection[P: PartialOrder](f: P -> P) {
    fixed_point_set(f) = pre_fixed_point_set(f).intersection(post_fixed_point_set(f))
} by {
    forall(x: P) {
        if fixed_point_set(f).contains(x) {
            fixed_point_set_subset_pre_fixed_point_set(f)
            subset_contains(fixed_point_set(f), pre_fixed_point_set(f), x)
            pre_fixed_point_set(f).contains(x)
            fixed_point_set_subset_post_fixed_point_set(f)
            subset_contains(fixed_point_set(f), post_fixed_point_set(f), x)
            post_fixed_point_set(f).contains(x)
            pre_fixed_point_set(f).intersection(post_fixed_point_set(f)).contains(x)
        }
        if pre_fixed_point_set(f).intersection(post_fixed_point_set(f)).contains(x) {
            pre_post_fixed_point_sets_subset_fixed_point_set(f)
            subset_contains(pre_fixed_point_set(f).intersection(post_fixed_point_set(f)), fixed_point_set(f), x)
            fixed_point_set(f).contains(x)
        }
        fixed_point_set(f).contains(x) =
            pre_fixed_point_set(f).intersection(post_fixed_point_set(f)).contains(x)
    }
    set_ext(fixed_point_set(f), pre_fixed_point_set(f).intersection(post_fixed_point_set(f)))
}

/// Every point is a fixed point of the identity map.
theorem identity_fixed_point[T](x: T) {
    is_fixed_point(function(y: T) { y }, x)
}

/// If `f(x) = x`, then `f` applied twice at `x` is still `x`.
theorem fixed_point_iterate[T](f: T -> T, x: T) {
    is_fixed_point(f, x) implies f(f(x)) = x
}

/// A monotone map sends pre-fixed points to pre-fixed points.
theorem monotone_pre_fixed_point_apply[P: PartialOrder](f: P -> P, x: P) {
    is_monotone(f) and is_pre_fixed_point(f, x) implies is_pre_fixed_point(f, f(x))
} by {
    if is_monotone(f) and is_pre_fixed_point(f, x) {
        f(x) <= x
        monotone_step(f, f(x), x)
        f(f(x)) <= f(x)
    }
}

/// A monotone map sends post-fixed points to post-fixed points.
theorem monotone_post_fixed_point_apply[P: PartialOrder](f: P -> P, x: P) {
    is_monotone(f) and is_post_fixed_point(f, x) implies is_post_fixed_point(f, f(x))
} by {
    if is_monotone(f) and is_post_fixed_point(f, x) {
        x <= f(x)
        monotone_step(f, x, f(x))
        f(x) <= f(f(x))
    }
}

/// A monotone map sends fixed points to fixed points.
theorem monotone_fixed_point_apply[P: PartialOrder](f: P -> P, x: P) {
    is_monotone(f) and is_fixed_point(f, x) implies is_fixed_point(f, f(x))
}

/// A closure operator makes every element post-fixed.
theorem closure_operator_post_fixed_point[P: PartialOrder](close: P -> P, x: P) {
    is_closure_operator(close) implies is_post_fixed_point(close, x)
} by {
    if is_closure_operator(close) {
        closure_operator_extensive_at(close, x)
        x <= close(x)
    }
}

/// A kernel operator makes every element pre-fixed.
theorem kernel_operator_pre_fixed_point[P: PartialOrder](kernel: P -> P, x: P) {
    is_kernel_operator(kernel) implies is_pre_fixed_point(kernel, x)
} by {
    if is_kernel_operator(kernel) {
        kernel_operator_reductive_at(kernel, x)
        kernel(x) <= x
    }
}

/// A closure operator sends every element to a fixed point.
theorem closure_operator_image_fixed_point[P: PartialOrder](close: P -> P, x: P) {
    is_closure_operator(close) implies is_fixed_point(close, close(x))
} by {
    if is_closure_operator(close) {
        closure_operator_idempotent_at(close, x)
        close(close(x)) = close(x)
        is_fixed_point(close, close(x))
    }
}

/// A kernel operator sends every element to a fixed point.
theorem kernel_operator_image_fixed_point[P: PartialOrder](kernel: P -> P, x: P) {
    is_kernel_operator(kernel) implies is_fixed_point(kernel, kernel(x))
} by {
    if is_kernel_operator(kernel) {
        kernel_operator_idempotent_at(kernel, x)
        kernel(kernel(x)) = kernel(x)
        is_fixed_point(kernel, kernel(x))
    }
}

/// The image of a closure operator belongs to its fixed-point set.
theorem closure_operator_image_mem_fixed_point_set[P: PartialOrder](close: P -> P, x: P) {
    is_closure_operator(close) implies fixed_point_set(close).contains(close(x))
} by {
    if is_closure_operator(close) {
        closure_operator_image_fixed_point(close, x)
        is_fixed_point(close, close(x))
        fixed_point_set_contains_eq(close, close(x))
        fixed_point_set(close).contains(close(x))
    }
}

/// The image of a kernel operator belongs to its fixed-point set.
theorem kernel_operator_image_mem_fixed_point_set[P: PartialOrder](kernel: P -> P, x: P) {
    is_kernel_operator(kernel) implies fixed_point_set(kernel).contains(kernel(x))
} by {
    if is_kernel_operator(kernel) {
        kernel_operator_image_fixed_point(kernel, x)
        is_fixed_point(kernel, kernel(x))
        fixed_point_set_contains_eq(kernel, kernel(x))
        fixed_point_set(kernel).contains(kernel(x))
    }
}

/// A pre-fixed point of a closure operator is a fixed point.
theorem closure_operator_fixed_point_of_pre_fixed[P: PartialOrder](close: P -> P, x: P) {
    is_closure_operator(close) and is_pre_fixed_point(close, x) implies is_fixed_point(close, x)
} by {
    if is_closure_operator(close) and is_pre_fixed_point(close, x) {
        close(x) <= x
        closure_operator_extensive_at(close, x)
        x <= close(x)
        lte_antisymm(close(x), x)
        close(x) = x
        is_fixed_point(close, x)
    }
}

/// A post-fixed point of a kernel operator is a fixed point.
theorem kernel_operator_fixed_point_of_post_fixed[P: PartialOrder](kernel: P -> P, x: P) {
    is_kernel_operator(kernel) and is_post_fixed_point(kernel, x) implies is_fixed_point(kernel, x)
} by {
    if is_kernel_operator(kernel) and is_post_fixed_point(kernel, x) {
        x <= kernel(x)
        kernel_operator_reductive_at(kernel, x)
        kernel(x) <= x
        lte_antisymm(kernel(x), x)
        kernel(x) = x
        is_fixed_point(kernel, x)
    }
}

/// For a closure operator, being pre-fixed is the same as being fixed.
theorem closure_operator_pre_fixed_iff_fixed_point[P: PartialOrder](close: P -> P, x: P) {
    is_closure_operator(close) implies (is_pre_fixed_point(close, x) = is_fixed_point(close, x))
} by {
    if is_closure_operator(close) {
        if is_pre_fixed_point(close, x) {
            closure_operator_fixed_point_of_pre_fixed(close, x)
            is_fixed_point(close, x)
        }
        if is_fixed_point(close, x) {
            fixed_point_is_pre_fixed(close, x)
            is_pre_fixed_point(close, x)
        }
        is_pre_fixed_point(close, x) = is_fixed_point(close, x)
    }
}

/// For a kernel operator, being post-fixed is the same as being fixed.
theorem kernel_operator_post_fixed_iff_fixed_point[P: PartialOrder](kernel: P -> P, x: P) {
    is_kernel_operator(kernel) implies (is_post_fixed_point(kernel, x) = is_fixed_point(kernel, x))
} by {
    if is_kernel_operator(kernel) {
        if is_post_fixed_point(kernel, x) {
            kernel_operator_fixed_point_of_post_fixed(kernel, x)
            is_fixed_point(kernel, x)
        }
        if is_fixed_point(kernel, x) {
            fixed_point_is_post_fixed(kernel, x)
            is_post_fixed_point(kernel, x)
        }
        is_post_fixed_point(kernel, x) = is_fixed_point(kernel, x)
    }
}

/// A closure operator maps its pre-fixed-point set into its fixed-point set.
theorem closure_operator_pre_fixed_set_subset_fixed_point_set[P: PartialOrder](close: P -> P) {
    is_closure_operator(close) implies pre_fixed_point_set(close).subset(fixed_point_set(close))
} by {
    if is_closure_operator(close) {
        forall(x: P) {
            if pre_fixed_point_set(close).contains(x) {
                pre_fixed_point_set_contains_eq(close, x)
                is_pre_fixed_point(close, x)
                closure_operator_fixed_point_of_pre_fixed(close, x)
                is_fixed_point(close, x)
                fixed_point_set_contains_eq(close, x)
                fixed_point_set(close).contains(x)
            }
        }
    }
}

/// A kernel operator maps its post-fixed-point set into its fixed-point set.
theorem kernel_operator_post_fixed_set_subset_fixed_point_set[P: PartialOrder](kernel: P -> P) {
    is_kernel_operator(kernel) implies post_fixed_point_set(kernel).subset(fixed_point_set(kernel))
} by {
    if is_kernel_operator(kernel) {
        forall(x: P) {
            if post_fixed_point_set(kernel).contains(x) {
                post_fixed_point_set_contains_eq(kernel, x)
                is_post_fixed_point(kernel, x)
                kernel_operator_fixed_point_of_post_fixed(kernel, x)
                is_fixed_point(kernel, x)
                fixed_point_set_contains_eq(kernel, x)
                fixed_point_set(kernel).contains(x)
            }
        }
    }
}

/// For a closure operator, fixed points are exactly pre-fixed points.
theorem closure_operator_fixed_point_set_eq_pre_fixed_point_set[P: PartialOrder](close: P -> P) {
    is_closure_operator(close) implies fixed_point_set(close) = pre_fixed_point_set(close)
} by {
    if is_closure_operator(close) {
        fixed_point_set_subset_pre_fixed_point_set(close)
        fixed_point_set(close).subset(pre_fixed_point_set(close))
        closure_operator_pre_fixed_set_subset_fixed_point_set(close)
        pre_fixed_point_set(close).subset(fixed_point_set(close))
        subset_antisymm(fixed_point_set(close), pre_fixed_point_set(close))
    }
}

/// For a kernel operator, fixed points are exactly post-fixed points.
theorem kernel_operator_fixed_point_set_eq_post_fixed_point_set[P: PartialOrder](kernel: P -> P) {
    is_kernel_operator(kernel) implies fixed_point_set(kernel) = post_fixed_point_set(kernel)
} by {
    if is_kernel_operator(kernel) {
        fixed_point_set_subset_post_fixed_point_set(kernel)
        fixed_point_set(kernel).subset(post_fixed_point_set(kernel))
        kernel_operator_post_fixed_set_subset_fixed_point_set(kernel)
        post_fixed_point_set(kernel).subset(fixed_point_set(kernel))
        subset_antisymm(fixed_point_set(kernel), post_fixed_point_set(kernel))
    }
}

/// The closure induced by a Galois connection makes every element post-fixed.
theorem galois_closure_post_fixed_point[A: PartialOrder, B: PartialOrder](
    lower: A -> B,
    upper: B -> A,
    a: A
) {
    is_galois_connection(lower, upper) implies is_post_fixed_point(galois_closure(lower, upper), a)
} by {
    if is_galois_connection(lower, upper) {
        galois_closure_extensive(lower, upper, a)
        a <= galois_closure(lower, upper, a)
        is_post_fixed_point(galois_closure(lower, upper), a)
    }
}

/// The kernel induced by a Galois connection makes every element pre-fixed.
theorem galois_kernel_pre_fixed_point[A: PartialOrder, B: PartialOrder](
    lower: A -> B,
    upper: B -> A,
    b: B
) {
    is_galois_connection(lower, upper) implies is_pre_fixed_point(galois_kernel(lower, upper), b)
} by {
    if is_galois_connection(lower, upper) {
        galois_kernel_reductive(lower, upper, b)
        galois_kernel(lower, upper, b) <= b
        is_pre_fixed_point(galois_kernel(lower, upper), b)
    }
}

/// The closure induced by a Galois connection sends every element to a fixed point.
theorem galois_closure_image_fixed_point[A: PartialOrder, B: PartialOrder](
    lower: A -> B,
    upper: B -> A,
    a: A
) {
    is_galois_connection(lower, upper) implies
    is_fixed_point(galois_closure(lower, upper), galois_closure(lower, upper, a))
} by {
    if is_galois_connection(lower, upper) {
        galois_closure_is_closure_operator(lower, upper)
        closure_operator_image_fixed_point(galois_closure(lower, upper), a)
        is_fixed_point(galois_closure(lower, upper), galois_closure(lower, upper, a))
    }
}

/// The kernel induced by a Galois connection sends every element to a fixed point.
theorem galois_kernel_image_fixed_point[A: PartialOrder, B: PartialOrder](
    lower: A -> B,
    upper: B -> A,
    b: B
) {
    is_galois_connection(lower, upper) implies
    is_fixed_point(galois_kernel(lower, upper), galois_kernel(lower, upper, b))
} by {
    if is_galois_connection(lower, upper) {
        galois_kernel_is_kernel_operator(lower, upper)
        kernel_operator_image_fixed_point(galois_kernel(lower, upper), b)
        is_fixed_point(galois_kernel(lower, upper), galois_kernel(lower, upper, b))
    }
}

/// For a Galois closure, being pre-fixed is the same as being fixed.
theorem galois_closure_pre_fixed_iff_fixed_point[A: PartialOrder, B: PartialOrder](
    lower: A -> B,
    upper: B -> A,
    a: A
) {
    is_galois_connection(lower, upper) implies
    (is_pre_fixed_point(galois_closure(lower, upper), a) =
        is_fixed_point(galois_closure(lower, upper), a))
} by {
    if is_galois_connection(lower, upper) {
        galois_closure_is_closure_operator(lower, upper)
        closure_operator_pre_fixed_iff_fixed_point(galois_closure(lower, upper), a)
        is_pre_fixed_point(galois_closure(lower, upper), a) =
            is_fixed_point(galois_closure(lower, upper), a)
    }
}

/// For a Galois kernel, being post-fixed is the same as being fixed.
theorem galois_kernel_post_fixed_iff_fixed_point[A: PartialOrder, B: PartialOrder](
    lower: A -> B,
    upper: B -> A,
    b: B
) {
    is_galois_connection(lower, upper) implies
    (is_post_fixed_point(galois_kernel(lower, upper), b) =
        is_fixed_point(galois_kernel(lower, upper), b))
} by {
    if is_galois_connection(lower, upper) {
        galois_kernel_is_kernel_operator(lower, upper)
        kernel_operator_post_fixed_iff_fixed_point(galois_kernel(lower, upper), b)
        is_post_fixed_point(galois_kernel(lower, upper), b) =
            is_fixed_point(galois_kernel(lower, upper), b)
    }
}

/// For a Galois closure, fixed points are exactly pre-fixed points.
theorem galois_closure_fixed_point_set_eq_pre_fixed_point_set[A: PartialOrder, B: PartialOrder](
    lower: A -> B,
    upper: B -> A
) {
    is_galois_connection(lower, upper) implies
    fixed_point_set(galois_closure(lower, upper)) =
        pre_fixed_point_set(galois_closure(lower, upper))
} by {
    if is_galois_connection(lower, upper) {
        galois_closure_is_closure_operator(lower, upper)
        closure_operator_fixed_point_set_eq_pre_fixed_point_set(galois_closure(lower, upper))
        fixed_point_set(galois_closure(lower, upper)) =
            pre_fixed_point_set(galois_closure(lower, upper))
    }
}

/// For a Galois kernel, fixed points are exactly post-fixed points.
theorem galois_kernel_fixed_point_set_eq_post_fixed_point_set[A: PartialOrder, B: PartialOrder](
    lower: A -> B,
    upper: B -> A
) {
    is_galois_connection(lower, upper) implies
    fixed_point_set(galois_kernel(lower, upper)) =
        post_fixed_point_set(galois_kernel(lower, upper))
} by {
    if is_galois_connection(lower, upper) {
        galois_kernel_is_kernel_operator(lower, upper)
        kernel_operator_fixed_point_set_eq_post_fixed_point_set(galois_kernel(lower, upper))
        fixed_point_set(galois_kernel(lower, upper)) =
            post_fixed_point_set(galois_kernel(lower, upper))
    }
}

/// If `f` is monotone, `a` is post-fixed, `b` is pre-fixed, and `a <= b`, then `f(a) <= f(b)` keeps the sandwich.
theorem monotone_post_le_pre[P: PartialOrder](f: P -> P, a: P, b: P) {
    is_monotone(f) and is_post_fixed_point(f, a) and is_pre_fixed_point(f, b) and a <= b
        implies f(a) <= f(b)
} by {
    if is_monotone(f) and is_post_fixed_point(f, a) and is_pre_fixed_point(f, b) and a <= b {
        monotone_step(f, a, b)
    }
}

/// A monotone map keeps every iterate of a pre-fixed point below the original point.
theorem monotone_iterate_pre_fixed[P: PartialOrder](f: P -> P, x: P, n: Nat) {
    is_monotone(f) and is_pre_fixed_point(f, x) implies iterate(f, n, x) <= x
} by {
    if is_monotone(f) and is_pre_fixed_point(f, x) {
        define p(k: Nat) -> Bool {
            iterate(f, k, x) <= x
        }
        iterate(f, Nat.0, x) = x
        lte_refl(x)
        p(Nat.0)
        forall(k: Nat) {
            if p(k) {
                iterate(f, k, x) <= x
                monotone_step(f, iterate(f, k, x), x)
                f(iterate(f, k, x)) <= f(x)
                iterate(f, k.suc, x) = f(iterate(f, k, x))
                iterate(f, k.suc, x) <= f(x)
                f(x) <= x
                lte_trans(iterate(f, k.suc, x), f(x), x)
                p(k.suc)
            }
        }
        p(n)
        iterate(f, n, x) <= x
    }
}

/// A monotone map keeps every iterate of a post-fixed point above the original point.
theorem monotone_iterate_post_fixed[P: PartialOrder](f: P -> P, x: P, n: Nat) {
    is_monotone(f) and is_post_fixed_point(f, x) implies x <= iterate(f, n, x)
} by {
    if is_monotone(f) and is_post_fixed_point(f, x) {
        define p(k: Nat) -> Bool {
            x <= iterate(f, k, x)
        }
        iterate(f, Nat.0, x) = x
        lte_refl(x)
        p(Nat.0)
        forall(k: Nat) {
            if p(k) {
                x <= iterate(f, k, x)
                monotone_step(f, x, iterate(f, k, x))
                f(x) <= f(iterate(f, k, x))
                iterate(f, k.suc, x) = f(iterate(f, k, x))
                f(x) <= iterate(f, k.suc, x)
                x <= f(x)
                lte_trans(x, f(x), iterate(f, k.suc, x))
                p(k.suc)
            }
        }
        p(n)
        x <= iterate(f, n, x)
    }
}

/// A monotone map sends every iterate of a fixed point back to itself.
theorem monotone_iterate_fixed_point[P: PartialOrder](f: P -> P, x: P, n: Nat) {
    is_monotone(f) and is_fixed_point(f, x) implies iterate(f, n, x) = x
} by {
    if is_monotone(f) and is_fixed_point(f, x) {
        define p(k: Nat) -> Bool {
            iterate(f, k, x) = x
        }
        iterate(f, Nat.0, x) = x
        p(Nat.0)
        forall(k: Nat) {
            if p(k) {
                iterate(f, k, x) = x
                iterate(f, k.suc, x) = f(iterate(f, k, x))
                iterate(f, k.suc, x) = f(x)
                f(x) = x
                p(k.suc)
            }
        }
        p(n)
        iterate(f, n, x) = x
    }
}

/// Iterates of a pre-fixed point under a monotone map form a descending chain.
theorem monotone_iterate_pre_fixed_descending[P: PartialOrder](f: P -> P, x: P, n: Nat) {
    is_monotone(f) and is_pre_fixed_point(f, x) implies iterate(f, n.suc, x) <= iterate(f, n, x)
} by {
    if is_monotone(f) and is_pre_fixed_point(f, x) {
        define p(k: Nat) -> Bool {
            iterate(f, k.suc, x) <= iterate(f, k, x)
        }
        iterate(f, Nat.0, x) = x
        iterate(f, Nat.0.suc, x) = f(x)
        f(x) <= x
        p(Nat.0)
        forall(k: Nat) {
            if p(k) {
                iterate(f, k.suc, x) <= iterate(f, k, x)
                monotone_step(f, iterate(f, k.suc, x), iterate(f, k, x))
                f(iterate(f, k.suc, x)) <= f(iterate(f, k, x))
                iterate(f, k.suc.suc, x) = f(iterate(f, k.suc, x))
                iterate(f, k.suc, x) = f(iterate(f, k, x))
                p(k.suc)
            }
        }
        p(n)
        iterate(f, n.suc, x) <= iterate(f, n, x)
    }
}

/// Iterates of a post-fixed point under a monotone map form an ascending chain.
theorem monotone_iterate_post_fixed_ascending[P: PartialOrder](f: P -> P, x: P, n: Nat) {
    is_monotone(f) and is_post_fixed_point(f, x) implies iterate(f, n, x) <= iterate(f, n.suc, x)
} by {
    if is_monotone(f) and is_post_fixed_point(f, x) {
        define p(k: Nat) -> Bool {
            iterate(f, k, x) <= iterate(f, k.suc, x)
        }
        iterate(f, Nat.0, x) = x
        iterate(f, Nat.0.suc, x) = f(x)
        x <= f(x)
        p(Nat.0)
        forall(k: Nat) {
            if p(k) {
                iterate(f, k, x) <= iterate(f, k.suc, x)
                monotone_step(f, iterate(f, k, x), iterate(f, k.suc, x))
                f(iterate(f, k, x)) <= f(iterate(f, k.suc, x))
                iterate(f, k.suc, x) = f(iterate(f, k, x))
                iterate(f, k.suc.suc, x) = f(iterate(f, k.suc, x))
                p(k.suc)
            }
        }
        p(n)
        iterate(f, n, x) <= iterate(f, n.suc, x)
    }
}

/// Every iterate of a pre-fixed point under a monotone map is itself a pre-fixed point.
theorem monotone_pre_fixed_iterate[P: PartialOrder](f: P -> P, x: P, n: Nat) {
    is_monotone(f) and is_pre_fixed_point(f, x) implies is_pre_fixed_point(f, iterate(f, n, x))
} by {
    if is_monotone(f) and is_pre_fixed_point(f, x) {
        f(x) <= x
        define p(k: Nat) -> Bool {
            f(iterate(f, k, x)) <= iterate(f, k, x)
        }
        iterate(f, Nat.0, x) = x
        f(iterate(f, Nat.0, x)) = f(x)
        p(Nat.0)
        forall(k: Nat) {
            if p(k) {
                f(iterate(f, k, x)) <= iterate(f, k, x)
                monotone_step(f, f(iterate(f, k, x)), iterate(f, k, x))
                f(f(iterate(f, k, x))) <= f(iterate(f, k, x))
                iterate(f, k.suc, x) = f(iterate(f, k, x))
                f(iterate(f, k.suc, x)) = f(f(iterate(f, k, x)))
                p(k.suc)
            }
        }
        p(n)
        f(iterate(f, n, x)) <= iterate(f, n, x)
    }
}

/// Every iterate of a post-fixed point under a monotone map is itself a post-fixed point.
theorem monotone_post_fixed_iterate[P: PartialOrder](f: P -> P, x: P, n: Nat) {
    is_monotone(f) and is_post_fixed_point(f, x) implies is_post_fixed_point(f, iterate(f, n, x))
} by {
    if is_monotone(f) and is_post_fixed_point(f, x) {
        x <= f(x)
        define p(k: Nat) -> Bool {
            iterate(f, k, x) <= f(iterate(f, k, x))
        }
        iterate(f, Nat.0, x) = x
        f(iterate(f, Nat.0, x)) = f(x)
        p(Nat.0)
        forall(k: Nat) {
            if p(k) {
                iterate(f, k, x) <= f(iterate(f, k, x))
                monotone_step(f, iterate(f, k, x), f(iterate(f, k, x)))
                f(iterate(f, k, x)) <= f(f(iterate(f, k, x)))
                iterate(f, k.suc, x) = f(iterate(f, k, x))
                f(iterate(f, k.suc, x)) = f(f(iterate(f, k, x)))
                p(k.suc)
            }
        }
        p(n)
        iterate(f, n, x) <= f(iterate(f, n, x))
    }
}

/// Every iterate of a fixed point under a monotone map is itself a fixed point.
theorem monotone_fixed_point_iterate[P: PartialOrder](f: P -> P, x: P, n: Nat) {
    is_monotone(f) and is_fixed_point(f, x) implies is_fixed_point(f, iterate(f, n, x))
} by {
    if is_monotone(f) and is_fixed_point(f, x) {
        monotone_iterate_fixed_point(f, x, n)
        iterate(f, n, x) = x
    }
}

/// Iterating further on a pre-fixed point under a monotone map descends.
theorem monotone_iterate_pre_fixed_distance[P: PartialOrder](f: P -> P, x: P, m: Nat, k: Nat) {
    is_monotone(f) and is_pre_fixed_point(f, x) implies iterate(f, m + k, x) <= iterate(f, m, x)
} by {
    if is_monotone(f) and is_pre_fixed_point(f, x) {
        iterate_add(f, m, k, x)
        iterate(f, m + k, x) = iterate(f, m, iterate(f, k, x))
        monotone_iterate_pre_fixed(f, x, k)
        is_pre_fixed_point(f, iterate(f, k, x))
        monotone_iterate_pre_fixed(f, iterate(f, k, x), m)
        f(iterate(f, k, x)) <= iterate(f, k, x)
        // We instead need iterate(f, m, iterate(f, k, x)) <= iterate(f, m, x).
        // Use that iterate(f, k, x) <= x and m-fold monotone step.
        monotone_iterate_pre_fixed(f, x, k)
        iterate(f, k, x) <= x
        define q(j: Nat) -> Bool {
            iterate(f, j, iterate(f, k, x)) <= iterate(f, j, x)
        }
        iterate(f, Nat.0, iterate(f, k, x)) = iterate(f, k, x)
        iterate(f, Nat.0, x) = x
        q(Nat.0)
        forall(j: Nat) {
            if q(j) {
                iterate(f, j, iterate(f, k, x)) <= iterate(f, j, x)
                monotone_step(f, iterate(f, j, iterate(f, k, x)), iterate(f, j, x))
                f(iterate(f, j, iterate(f, k, x))) <= f(iterate(f, j, x))
                iterate(f, j.suc, iterate(f, k, x)) = f(iterate(f, j, iterate(f, k, x)))
                iterate(f, j.suc, x) = f(iterate(f, j, x))
                iterate(f, j.suc, iterate(f, k, x)) <= iterate(f, j.suc, x)
                q(j.suc)
            }
        }
        q(m)
        iterate(f, m, iterate(f, k, x)) <= iterate(f, m, x)
    }
}

/// A monotone map preserves order through equal-length iteration.
theorem monotone_iterate_le_of_le[P: PartialOrder](f: P -> P, a: P, b: P, n: Nat) {
    is_monotone(f) and a <= b implies iterate(f, n, a) <= iterate(f, n, b)
} by {
    if is_monotone(f) and a <= b {
        define p(k: Nat) -> Bool {
            iterate(f, k, a) <= iterate(f, k, b)
        }
        iterate(f, Nat.0, a) = a
        iterate(f, Nat.0, b) = b
        p(Nat.0)
        forall(k: Nat) {
            if p(k) {
                iterate(f, k, a) <= iterate(f, k, b)
                monotone_step(f, iterate(f, k, a), iterate(f, k, b))
                f(iterate(f, k, a)) <= f(iterate(f, k, b))
                iterate(f, k.suc, a) = f(iterate(f, k, a))
                iterate(f, k.suc, b) = f(iterate(f, k, b))
                p(k.suc)
            }
        }
        p(n)
        iterate(f, n, a) <= iterate(f, n, b)
    }
}

/// Equal-length iterates preserve an initial order relation.
theorem monotone_iterate_sandwich_middle[P: PartialOrder](f: P -> P, x: P, y: P, n: Nat) {
    is_monotone(f) and y <= x implies iterate(f, n, y) <= iterate(f, n, x)
} by {
    if is_monotone(f) and y <= x {
        monotone_iterate_le_of_le(f, y, x, n)
    }
}

/// Iterating further on a post-fixed point under a monotone map ascends.
theorem monotone_iterate_post_fixed_distance[P: PartialOrder](f: P -> P, x: P, m: Nat, k: Nat) {
    is_monotone(f) and is_post_fixed_point(f, x) implies iterate(f, m, x) <= iterate(f, m + k, x)
} by {
    if is_monotone(f) and is_post_fixed_point(f, x) {
        iterate_add(f, m, k, x)
        iterate(f, m + k, x) = iterate(f, m, iterate(f, k, x))
        monotone_iterate_post_fixed(f, x, k)
        x <= iterate(f, k, x)
        define q(j: Nat) -> Bool {
            iterate(f, j, x) <= iterate(f, j, iterate(f, k, x))
        }
        iterate(f, Nat.0, iterate(f, k, x)) = iterate(f, k, x)
        iterate(f, Nat.0, x) = x
        q(Nat.0)
        forall(j: Nat) {
            if q(j) {
                iterate(f, j, x) <= iterate(f, j, iterate(f, k, x))
                monotone_step(f, iterate(f, j, x), iterate(f, j, iterate(f, k, x)))
                f(iterate(f, j, x)) <= f(iterate(f, j, iterate(f, k, x)))
                iterate(f, j.suc, iterate(f, k, x)) = f(iterate(f, j, iterate(f, k, x)))
                iterate(f, j.suc, x) = f(iterate(f, j, x))
                iterate(f, j.suc, x) <= iterate(f, j.suc, iterate(f, k, x))
                q(j.suc)
            }
        }
        q(m)
        iterate(f, m, x) <= iterate(f, m, iterate(f, k, x))
    }
}

/// Iterates from a post-fixed lower bound stay below matching iterates from a larger point.
theorem monotone_post_fixed_iterate_le_of_le[P: PartialOrder](f: P -> P, a: P, x: P, n: Nat) {
    is_monotone(f) and is_post_fixed_point(f, a) and a <= x
    implies a <= iterate(f, n, x)
} by {
    if is_monotone(f) and is_post_fixed_point(f, a) and a <= x {
        monotone_iterate_post_fixed(f, a, n)
        a <= iterate(f, n, a)
        monotone_iterate_le_of_le(f, a, x, n)
        iterate(f, n, a) <= iterate(f, n, x)
        lte_trans(a, iterate(f, n, a), iterate(f, n, x))
        a <= iterate(f, n, x)
    }
}

/// Iterates from a point below a pre-fixed upper bound stay below that bound.
theorem monotone_iterate_le_pre_fixed_of_le[P: PartialOrder](f: P -> P, x: P, b: P, n: Nat) {
    is_monotone(f) and is_pre_fixed_point(f, b) and x <= b
    implies iterate(f, n, x) <= b
} by {
    if is_monotone(f) and is_pre_fixed_point(f, b) and x <= b {
        monotone_iterate_le_of_le(f, x, b, n)
        iterate(f, n, x) <= iterate(f, n, b)
        monotone_iterate_pre_fixed(f, b, n)
        iterate(f, n, b) <= b
        lte_trans(iterate(f, n, x), iterate(f, n, b), b)
        iterate(f, n, x) <= b
    }
}

/// Iterates remain between a post-fixed lower bound and a pre-fixed upper bound.
theorem monotone_iterate_between_post_and_pre[P: PartialOrder](
    f: P -> P,
    a: P,
    x: P,
    b: P,
    n: Nat
) {
    is_monotone(f) and is_post_fixed_point(f, a) and is_pre_fixed_point(f, b) and
    a <= x and x <= b
    implies a <= iterate(f, n, x) and iterate(f, n, x) <= b
} by {
    if is_monotone(f) and is_post_fixed_point(f, a) and is_pre_fixed_point(f, b) and
        a <= x and x <= b {
        monotone_post_fixed_iterate_le_of_le(f, a, x, n)
        a <= iterate(f, n, x)
        monotone_iterate_le_pre_fixed_of_le(f, x, b, n)
        iterate(f, n, x) <= b
        a <= iterate(f, n, x) and iterate(f, n, x) <= b
    }
}

/// A post-fixed point below a pre-fixed point bounds every iterate from below.
theorem monotone_post_fixed_le_pre_fixed_iterate_lower[P: PartialOrder](
    f: P -> P,
    a: P,
    b: P,
    n: Nat
) {
    is_monotone(f) and is_post_fixed_point(f, a) and is_pre_fixed_point(f, b) and a <= b
    implies a <= iterate(f, n, b)
} by {
    if is_monotone(f) and is_post_fixed_point(f, a) and is_pre_fixed_point(f, b) and a <= b {
        monotone_post_fixed_iterate_le_of_le(f, a, b, n)
        a <= iterate(f, n, b)
    }
}

/// A post-fixed point below a pre-fixed point bounds every iterate from above.
theorem monotone_post_fixed_le_pre_fixed_iterate_upper[P: PartialOrder](
    f: P -> P,
    a: P,
    b: P,
    n: Nat
) {
    is_monotone(f) and is_post_fixed_point(f, a) and is_pre_fixed_point(f, b) and a <= b
    implies iterate(f, n, a) <= b
} by {
    if is_monotone(f) and is_post_fixed_point(f, a) and is_pre_fixed_point(f, b) and a <= b {
        monotone_iterate_le_pre_fixed_of_le(f, a, b, n)
        iterate(f, n, a) <= b
    }
}

/// Iterates of a pre-fixed point under a monotone map are antitone in the iteration index.
theorem monotone_iterate_pre_fixed_anti_index[P: PartialOrder](
    f: P -> P,
    x: P,
    k: Nat,
    m: Nat
) {
    is_monotone(f) and is_pre_fixed_point(f, x) and k <= m
    implies iterate(f, m, x) <= iterate(f, k, x)
} by {
    if is_monotone(f) and is_pre_fixed_point(f, x) and k <= m {
        let d: Nat satisfy { k + d = m }
        monotone_iterate_pre_fixed_distance(f, x, k, d)
        iterate(f, k + d, x) <= iterate(f, k, x)
        iterate(f, m, x) <= iterate(f, k, x)
    }
}

/// Iterates of a post-fixed point under a monotone map are monotone in the iteration index.
theorem monotone_iterate_post_fixed_mono_index[P: PartialOrder](
    f: P -> P,
    x: P,
    k: Nat,
    m: Nat
) {
    is_monotone(f) and is_post_fixed_point(f, x) and k <= m
    implies iterate(f, k, x) <= iterate(f, m, x)
} by {
    if is_monotone(f) and is_post_fixed_point(f, x) and k <= m {
        let d: Nat satisfy { k + d = m }
        monotone_iterate_post_fixed_distance(f, x, k, d)
        iterate(f, k, x) <= iterate(f, k + d, x)
        iterate(f, k, x) <= iterate(f, m, x)
    }
}

/// Every iterate of a fixed point under any map is the point itself.
theorem fixed_point_iterate_eq[T](f: T -> T, x: T, n: Nat) {
    is_fixed_point(f, x) implies iterate(f, n, x) = x
} by {
    if is_fixed_point(f, x) {
        define p(k: Nat) -> Bool {
            iterate(f, k, x) = x
        }
        iterate(f, Nat.0, x) = x
        p(Nat.0)
        forall(k: Nat) {
            if p(k) {
                iterate(f, k, x) = x
                iterate(f, k.suc, x) = f(iterate(f, k, x))
                iterate(f, k.suc, x) = f(x)
                f(x) = x
                p(k.suc)
            }
        }
        p(n)
        iterate(f, n, x) = x
    }
}

/// Iterates from a fixed point under any map agree at every pair of indices.
theorem fixed_point_iterate_eq_iterate[T](f: T -> T, x: T, m: Nat, n: Nat) {
    is_fixed_point(f, x) implies iterate(f, m, x) = iterate(f, n, x)
} by {
    if is_fixed_point(f, x) {
        fixed_point_iterate_eq(f, x, m)
        iterate(f, m, x) = x
        fixed_point_iterate_eq(f, x, n)
        iterate(f, n, x) = x
        iterate(f, m, x) = iterate(f, n, x)
    }
}

/// A monotone map maps the fixed-point set to itself pointwise.
theorem monotone_fixed_point_set_apply[P: PartialOrder](f: P -> P, x: P) {
    is_monotone(f) and fixed_point_set(f).contains(x) implies fixed_point_set(f).contains(f(x))
} by {
    if is_monotone(f) and fixed_point_set(f).contains(x) {
        fixed_point_set_contains_eq(f, x)
        is_fixed_point(f, x)
        monotone_fixed_point_apply(f, x)
        is_fixed_point(f, f(x))
        fixed_point_set_contains_eq(f, f(x))
        fixed_point_set(f).contains(f(x))
    }
}

/// A monotone map maps the pre-fixed-point set to itself pointwise.
theorem monotone_pre_fixed_point_set_apply[P: PartialOrder](f: P -> P, x: P) {
    is_monotone(f) and pre_fixed_point_set(f).contains(x) implies pre_fixed_point_set(f).contains(f(x))
} by {
    if is_monotone(f) and pre_fixed_point_set(f).contains(x) {
        pre_fixed_point_set_contains_eq(f, x)
        is_pre_fixed_point(f, x)
        monotone_pre_fixed_point_apply(f, x)
        is_pre_fixed_point(f, f(x))
        pre_fixed_point_set_contains_eq(f, f(x))
        pre_fixed_point_set(f).contains(f(x))
    }
}

/// A monotone map maps the post-fixed-point set to itself pointwise.
theorem monotone_post_fixed_point_set_apply[P: PartialOrder](f: P -> P, x: P) {
    is_monotone(f) and post_fixed_point_set(f).contains(x) implies post_fixed_point_set(f).contains(f(x))
} by {
    if is_monotone(f) and post_fixed_point_set(f).contains(x) {
        post_fixed_point_set_contains_eq(f, x)
        is_post_fixed_point(f, x)
        monotone_post_fixed_point_apply(f, x)
        is_post_fixed_point(f, f(x))
        post_fixed_point_set_contains_eq(f, f(x))
        post_fixed_point_set(f).contains(f(x))
    }
}

/// Iterates of a fixed point of a monotone map remain in the fixed-point set.
theorem monotone_fixed_point_set_iterate[P: PartialOrder](f: P -> P, x: P, n: Nat) {
    is_monotone(f) and fixed_point_set(f).contains(x)
        implies fixed_point_set(f).contains(iterate(f, n, x))
} by {
    if is_monotone(f) and fixed_point_set(f).contains(x) {
        fixed_point_set_contains_eq(f, x)
        is_fixed_point(f, x)
        monotone_fixed_point_iterate(f, x, n)
        is_fixed_point(f, iterate(f, n, x))
        fixed_point_set_contains_eq(f, iterate(f, n, x))
        fixed_point_set(f).contains(iterate(f, n, x))
    }
}

/// Iterates of a pre-fixed point of a monotone map remain in the pre-fixed-point set.
theorem monotone_pre_fixed_point_set_iterate[P: PartialOrder](f: P -> P, x: P, n: Nat) {
    is_monotone(f) and pre_fixed_point_set(f).contains(x)
        implies pre_fixed_point_set(f).contains(iterate(f, n, x))
} by {
    if is_monotone(f) and pre_fixed_point_set(f).contains(x) {
        pre_fixed_point_set_contains_eq(f, x)
        is_pre_fixed_point(f, x)
        monotone_pre_fixed_iterate(f, x, n)
        is_pre_fixed_point(f, iterate(f, n, x))
        pre_fixed_point_set_contains_eq(f, iterate(f, n, x))
        pre_fixed_point_set(f).contains(iterate(f, n, x))
    }
}

/// Iterates of a post-fixed point of a monotone map remain in the post-fixed-point set.
theorem monotone_post_fixed_point_set_iterate[P: PartialOrder](f: P -> P, x: P, n: Nat) {
    is_monotone(f) and post_fixed_point_set(f).contains(x)
        implies post_fixed_point_set(f).contains(iterate(f, n, x))
} by {
    if is_monotone(f) and post_fixed_point_set(f).contains(x) {
        post_fixed_point_set_contains_eq(f, x)
        is_post_fixed_point(f, x)
        monotone_post_fixed_iterate(f, x, n)
        is_post_fixed_point(f, iterate(f, n, x))
        post_fixed_point_set_contains_eq(f, iterate(f, n, x))
        post_fixed_point_set(f).contains(iterate(f, n, x))
    }
}

/// Applying a map to a pre-fixed point produces a pre-fixed point.
theorem monotone_apply_pre_fixed_is_pre_fixed[P: PartialOrder](f: P -> P, x: P) {
    is_monotone(f) and is_pre_fixed_point(f, x) implies is_pre_fixed_point(f, f(x))
} by {
    if is_monotone(f) and is_pre_fixed_point(f, x) {
        monotone_pre_fixed_point_apply(f, x)
    }
}

/// Applying a map to a post-fixed point produces a post-fixed point.
theorem monotone_apply_post_fixed_is_post_fixed[P: PartialOrder](f: P -> P, x: P) {
    is_monotone(f) and is_post_fixed_point(f, x) implies is_post_fixed_point(f, f(x))
} by {
    if is_monotone(f) and is_post_fixed_point(f, x) {
        monotone_post_fixed_point_apply(f, x)
    }
}

/// Under a monotone map, iterates of a pre-fixed point at larger indices lie below those at smaller indices.
theorem monotone_iterate_pre_fixed_le_lower_index[P: PartialOrder](
    f: P -> P,
    x: P,
    k: Nat,
    m: Nat
) {
    is_monotone(f) and is_pre_fixed_point(f, x) and k <= m
    implies iterate(f, m, x) <= iterate(f, k, x)
} by {
    if is_monotone(f) and is_pre_fixed_point(f, x) and k <= m {
        monotone_iterate_pre_fixed_anti_index(f, x, k, m)
    }
}

/// Under a monotone map, iterates of a post-fixed point at smaller indices lie below those at larger indices.
theorem monotone_iterate_post_fixed_le_higher_index[P: PartialOrder](
    f: P -> P,
    x: P,
    k: Nat,
    m: Nat
) {
    is_monotone(f) and is_post_fixed_point(f, x) and k <= m
    implies iterate(f, k, x) <= iterate(f, m, x)
} by {
    if is_monotone(f) and is_post_fixed_point(f, x) and k <= m {
        monotone_iterate_post_fixed_mono_index(f, x, k, m)
    }
}

/// Iterates of a pre-fixed point under a monotone map remain below the map's value at the point.
theorem monotone_iterate_pre_fixed_le_f[P: PartialOrder](f: P -> P, x: P, n: Nat) {
    is_monotone(f) and is_pre_fixed_point(f, x) implies iterate(f, n.suc, x) <= f(x)
} by {
    if is_monotone(f) and is_pre_fixed_point(f, x) {
        monotone_pre_fixed_iterate(f, x, n)
        is_pre_fixed_point(f, iterate(f, n, x))
        monotone_iterate_pre_fixed(f, x, n)
        iterate(f, n, x) <= x
        monotone_step(f, iterate(f, n, x), x)
        f(iterate(f, n, x)) <= f(x)
        iterate(f, n.suc, x) = f(iterate(f, n, x))
        iterate(f, n.suc, x) <= f(x)
    }
}

/// Iterates of a post-fixed point under a monotone map remain above the map's value at the point.
theorem monotone_iterate_post_fixed_ge_f[P: PartialOrder](f: P -> P, x: P, n: Nat) {
    is_monotone(f) and is_post_fixed_point(f, x) implies f(x) <= iterate(f, n.suc, x)
} by {
    if is_monotone(f) and is_post_fixed_point(f, x) {
        monotone_post_fixed_iterate(f, x, n)
        is_post_fixed_point(f, iterate(f, n, x))
        monotone_iterate_post_fixed(f, x, n)
        x <= iterate(f, n, x)
        monotone_step(f, x, iterate(f, n, x))
        f(x) <= f(iterate(f, n, x))
        iterate(f, n.suc, x) = f(iterate(f, n, x))
        f(x) <= iterate(f, n.suc, x)
    }
}

/// Under a monotone map, the iterates of a pre-fixed point sandwiched between two indices are themselves comparable.
theorem monotone_iterate_pre_fixed_sandwich[P: PartialOrder](
    f: P -> P,
    x: P,
    i: Nat,
    j: Nat,
    n: Nat
) {
    is_monotone(f) and is_pre_fixed_point(f, x) and i <= j and j <= n
    implies iterate(f, n, x) <= iterate(f, i, x)
} by {
    if is_monotone(f) and is_pre_fixed_point(f, x) and i <= j and j <= n {
        monotone_iterate_pre_fixed_anti_index(f, x, j, n)
        iterate(f, n, x) <= iterate(f, j, x)
        monotone_iterate_pre_fixed_anti_index(f, x, i, j)
        iterate(f, j, x) <= iterate(f, i, x)
        lte_trans(iterate(f, n, x), iterate(f, j, x), iterate(f, i, x))
        iterate(f, n, x) <= iterate(f, i, x)
    }
}

/// Under a monotone map, the iterates of a post-fixed point sandwiched between two indices are themselves comparable.
theorem monotone_iterate_post_fixed_sandwich[P: PartialOrder](
    f: P -> P,
    x: P,
    i: Nat,
    j: Nat,
    n: Nat
) {
    is_monotone(f) and is_post_fixed_point(f, x) and i <= j and j <= n
    implies iterate(f, i, x) <= iterate(f, n, x)
} by {
    if is_monotone(f) and is_post_fixed_point(f, x) and i <= j and j <= n {
        monotone_iterate_post_fixed_mono_index(f, x, i, j)
        iterate(f, i, x) <= iterate(f, j, x)
        monotone_iterate_post_fixed_mono_index(f, x, j, n)
        iterate(f, j, x) <= iterate(f, n, x)
        lte_trans(iterate(f, i, x), iterate(f, j, x), iterate(f, n, x))
        iterate(f, i, x) <= iterate(f, n, x)
    }
}

/// A fixed point of a closure operator agrees with its value under the operator.
theorem closure_operator_fixed_point_eq[P: PartialOrder](close: P -> P, x: P) {
    is_closure_operator(close) and is_fixed_point(close, x) implies close(x) = x
} by {
    if is_closure_operator(close) and is_fixed_point(close, x) {
        is_fixed_point(close, x) = (close(x) = x)
    }
}

/// A fixed point of a kernel operator agrees with its value under the operator.
theorem kernel_operator_fixed_point_eq[P: PartialOrder](kernel: P -> P, x: P) {
    is_kernel_operator(kernel) and is_fixed_point(kernel, x) implies kernel(x) = x
} by {
    if is_kernel_operator(kernel) and is_fixed_point(kernel, x) {
        is_fixed_point(kernel, x) = (kernel(x) = x)
    }
}

/// A point dominated by every pre-fixed point of a monotone map is dominated by all iterates of those points.
theorem monotone_iterate_lower_of_le_pre_fixed[P: PartialOrder](
    f: P -> P,
    a: P,
    b: P,
    n: Nat
) {
    is_monotone(f) and is_pre_fixed_point(f, b) and a <= b
    implies iterate(f, n, a) <= iterate(f, n, b) and iterate(f, n, b) <= b
} by {
    if is_monotone(f) and is_pre_fixed_point(f, b) and a <= b {
        monotone_iterate_le_of_le(f, a, b, n)
        iterate(f, n, a) <= iterate(f, n, b)
        monotone_iterate_pre_fixed(f, b, n)
        iterate(f, n, b) <= b
    }
}

/// The fixed-point set is closed under the action of a monotone map.
theorem monotone_fixed_point_set_closed[P: PartialOrder](f: P -> P) {
    is_monotone(f) implies forall(x: P) {
        fixed_point_set(f).contains(x) implies fixed_point_set(f).contains(f(x))
    }
} by {
    if is_monotone(f) {
        forall(x: P) {
            if fixed_point_set(f).contains(x) {
                monotone_fixed_point_set_apply(f, x)
                fixed_point_set(f).contains(f(x))
            }
        }
    }
}

/// The pre-fixed-point set is closed under the action of a monotone map.
theorem monotone_pre_fixed_point_set_closed[P: PartialOrder](f: P -> P) {
    is_monotone(f) implies forall(x: P) {
        pre_fixed_point_set(f).contains(x) implies pre_fixed_point_set(f).contains(f(x))
    }
} by {
    if is_monotone(f) {
        forall(x: P) {
            if pre_fixed_point_set(f).contains(x) {
                monotone_pre_fixed_point_set_apply(f, x)
                pre_fixed_point_set(f).contains(f(x))
            }
        }
    }
}

/// The post-fixed-point set is closed under the action of a monotone map.
theorem monotone_post_fixed_point_set_closed[P: PartialOrder](f: P -> P) {
    is_monotone(f) implies forall(x: P) {
        post_fixed_point_set(f).contains(x) implies post_fixed_point_set(f).contains(f(x))
    }
} by {
    if is_monotone(f) {
        forall(x: P) {
            if post_fixed_point_set(f).contains(x) {
                monotone_post_fixed_point_set_apply(f, x)
                post_fixed_point_set(f).contains(f(x))
            }
        }
    }
}

/// A common fixed point of two self-maps is a fixed point of their composition.
theorem fixed_point_of_compose_of_common[T](f: T -> T, g: T -> T, x: T) {
    is_fixed_point(f, x) and is_fixed_point(g, x) implies is_fixed_point(compose(f, g), x)
} by {
    if is_fixed_point(f, x) and is_fixed_point(g, x) {
        g(x) = x
        f(x) = x
        compose(f, g)(x) = f(g(x))
        compose(f, g)(x) = x
    }
}

/// A fixed point of a self-map is a fixed point of the composition of that map with itself.
theorem fixed_point_of_compose_self[T](f: T -> T, x: T) {
    is_fixed_point(f, x) implies is_fixed_point(compose(f, f), x)
} by {
    if is_fixed_point(f, x) {
        fixed_point_of_compose_of_common(f, f, x)
    }
}

/// When the inner map fixes a point, fixed points of the composition are fixed points of the outer map.
theorem fixed_point_of_outer_of_compose[T](f: T -> T, g: T -> T, x: T) {
    is_fixed_point(g, x) and is_fixed_point(compose(f, g), x) implies is_fixed_point(f, x)
} by {
    if is_fixed_point(g, x) and is_fixed_point(compose(f, g), x) {
        g(x) = x
        compose(f, g)(x) = f(g(x))
        compose(f, g)(x) = f(x)
        f(x) = x
    }
}

/// Composing with the identity on the left does not change the fixed points.
theorem fixed_point_compose_identity_left[T](f: T -> T, x: T) {
    is_fixed_point(compose(identity_fn[T], f), x) = is_fixed_point(f, x)
} by {
    compose(identity_fn[T], f)(x) = identity_fn[T](f(x))
    identity_fn[T](f(x)) = f(x)
    compose(identity_fn[T], f)(x) = f(x)
}

/// Composing with the identity on the right does not change the fixed points.
theorem fixed_point_compose_identity_right[T](f: T -> T, x: T) {
    is_fixed_point(compose(f, identity_fn[T]), x) = is_fixed_point(f, x)
} by {
    compose(f, identity_fn[T])(x) = f(identity_fn[T](x))
    identity_fn[T](x) = x
    compose(f, identity_fn[T])(x) = f(x)
}

/// The set of points fixed by both of two self-maps.
define common_fixed_point_set[T](f: T -> T, g: T -> T) -> Set[T] {
    fixed_point_set(f).intersection(fixed_point_set(g))
}

/// Membership in the common fixed-point set is being fixed by both maps.
theorem common_fixed_point_set_contains_eq[T](f: T -> T, g: T -> T, x: T) {
    common_fixed_point_set(f, g).contains(x) = (is_fixed_point(f, x) and is_fixed_point(g, x))
} by {
    common_fixed_point_set(f, g).contains(x) =
        elem_in_intersection(fixed_point_set(f), fixed_point_set(g), x)
    elem_in_intersection(fixed_point_set(f), fixed_point_set(g), x) =
        (fixed_point_set(f).contains(x) and fixed_point_set(g).contains(x))
    fixed_point_set_contains_eq(f, x)
    fixed_point_set_contains_eq(g, x)
}

/// The common fixed-point set is symmetric in its two maps.
theorem common_fixed_point_set_comm[T](f: T -> T, g: T -> T) {
    common_fixed_point_set(f, g) = common_fixed_point_set(g, f)
} by {
    forall(x: T) {
        common_fixed_point_set_contains_eq(f, g, x)
        common_fixed_point_set_contains_eq(g, f, x)
        common_fixed_point_set(f, g).contains(x) = common_fixed_point_set(g, f).contains(x)
    }
    set_ext(common_fixed_point_set(f, g), common_fixed_point_set(g, f))
}

/// The common fixed-point set is contained in the fixed-point set of each map.
theorem common_fixed_point_set_subset_left[T](f: T -> T, g: T -> T) {
    common_fixed_point_set(f, g).subset(fixed_point_set(f))
} by {
    forall(x: T) {
        if common_fixed_point_set(f, g).contains(x) {
            common_fixed_point_set_contains_eq(f, g, x)
            is_fixed_point(f, x)
            fixed_point_set_contains_eq(f, x)
            fixed_point_set(f).contains(x)
        }
    }
}

/// The common fixed-point set is contained in the fixed-point set of the second map.
theorem common_fixed_point_set_subset_right[T](f: T -> T, g: T -> T) {
    common_fixed_point_set(f, g).subset(fixed_point_set(g))
} by {
    common_fixed_point_set_comm(f, g)
    common_fixed_point_set_subset_left(g, f)
}

/// Every point is a fixed point of the identity map.
theorem identity_is_fixed_point[T](x: T) {
    is_fixed_point(identity_fn[T], x)
} by {
    identity_fn[T](x) = x
}

/// Every point is a pre-fixed point of the identity map.
theorem identity_is_pre_fixed_point[P: PartialOrder](x: P) {
    is_pre_fixed_point(identity_fn[P], x)
} by {
    identity_is_fixed_point(x)
    fixed_point_is_pre_fixed(identity_fn[P], x)
}

/// Every point is a post-fixed point of the identity map.
theorem identity_is_post_fixed_point[P: PartialOrder](x: P) {
    is_post_fixed_point(identity_fn[P], x)
} by {
    identity_is_fixed_point(x)
    fixed_point_is_post_fixed(identity_fn[P], x)
}

/// The common fixed points of a map with itself are exactly its fixed points.
theorem common_fixed_point_set_self[T](f: T -> T) {
    common_fixed_point_set(f, f) = fixed_point_set(f)
} by {
    forall(x: T) {
        common_fixed_point_set_contains_eq(f, f, x)
        fixed_point_set_contains_eq(f, x)
        common_fixed_point_set(f, f).contains(x) = fixed_point_set(f).contains(x)
    }
    set_ext(common_fixed_point_set(f, f), fixed_point_set(f))
}

/// Pairing a map with the identity on the left leaves the common fixed points unchanged.
theorem common_fixed_point_set_identity_left[T](g: T -> T) {
    common_fixed_point_set(identity_fn[T], g) = fixed_point_set(g)
} by {
    forall(x: T) {
        common_fixed_point_set_contains_eq(identity_fn[T], g, x)
        common_fixed_point_set(identity_fn[T], g).contains(x) =
            (is_fixed_point(identity_fn[T], x) and is_fixed_point(g, x))
        identity_is_fixed_point(x)
        common_fixed_point_set(identity_fn[T], g).contains(x) = is_fixed_point(g, x)
        fixed_point_set_contains_eq(g, x)
        common_fixed_point_set(identity_fn[T], g).contains(x) = fixed_point_set(g).contains(x)
    }
    set_ext(common_fixed_point_set(identity_fn[T], g), fixed_point_set(g))
}

/// Pairing a map with the identity on the right leaves the common fixed points unchanged.
theorem common_fixed_point_set_identity_right[T](f: T -> T) {
    common_fixed_point_set(f, identity_fn[T]) = fixed_point_set(f)
} by {
    common_fixed_point_set_comm(f, identity_fn[T])
    common_fixed_point_set_identity_left(f)
}

/// Under two monotone maps, a common pre-fixed point is a pre-fixed point of the composition.
theorem pre_fixed_point_of_compose[P: PartialOrder](f: P -> P, g: P -> P, x: P) {
    is_monotone(f) and is_pre_fixed_point(f, x) and is_pre_fixed_point(g, x)
    implies is_pre_fixed_point(compose(f, g), x)
} by {
    if is_monotone(f) and is_pre_fixed_point(f, x) and is_pre_fixed_point(g, x) {
        g(x) <= x
        monotone_step(f, g(x), x)
        f(g(x)) <= f(x)
        f(x) <= x
        lte_trans(f(g(x)), f(x), x)
        compose(f, g)(x) = f(g(x))
        compose(f, g)(x) <= x
    }
}

/// Under two monotone maps, a common post-fixed point is a post-fixed point of the composition.
theorem post_fixed_point_of_compose[P: PartialOrder](f: P -> P, g: P -> P, x: P) {
    is_monotone(f) and is_post_fixed_point(f, x) and is_post_fixed_point(g, x)
    implies is_post_fixed_point(compose(f, g), x)
} by {
    if is_monotone(f) and is_post_fixed_point(f, x) and is_post_fixed_point(g, x) {
        x <= g(x)
        monotone_step(f, x, g(x))
        f(x) <= f(g(x))
        x <= f(x)
        lte_trans(x, f(x), f(g(x)))
        compose(f, g)(x) = f(g(x))
        x <= compose(f, g)(x)
    }
}

/// The set of points pre-fixed by both of two self-maps.
define common_pre_fixed_point_set[P: PartialOrder](f: P -> P, g: P -> P) -> Set[P] {
    pre_fixed_point_set(f).intersection(pre_fixed_point_set(g))
}

/// The set of points post-fixed by both of two self-maps.
define common_post_fixed_point_set[P: PartialOrder](f: P -> P, g: P -> P) -> Set[P] {
    post_fixed_point_set(f).intersection(post_fixed_point_set(g))
}

/// Membership in the common pre-fixed-point set is being pre-fixed by both maps.
theorem common_pre_fixed_point_set_contains_eq[P: PartialOrder](f: P -> P, g: P -> P, x: P) {
    common_pre_fixed_point_set(f, g).contains(x) =
        (is_pre_fixed_point(f, x) and is_pre_fixed_point(g, x))
} by {
    common_pre_fixed_point_set(f, g).contains(x) =
        elem_in_intersection(pre_fixed_point_set(f), pre_fixed_point_set(g), x)
    elem_in_intersection(pre_fixed_point_set(f), pre_fixed_point_set(g), x) =
        (pre_fixed_point_set(f).contains(x) and pre_fixed_point_set(g).contains(x))
    pre_fixed_point_set_contains_eq(f, x)
    pre_fixed_point_set_contains_eq(g, x)
}

/// Membership in the common post-fixed-point set is being post-fixed by both maps.
theorem common_post_fixed_point_set_contains_eq[P: PartialOrder](f: P -> P, g: P -> P, x: P) {
    common_post_fixed_point_set(f, g).contains(x) =
        (is_post_fixed_point(f, x) and is_post_fixed_point(g, x))
} by {
    common_post_fixed_point_set(f, g).contains(x) =
        elem_in_intersection(post_fixed_point_set(f), post_fixed_point_set(g), x)
    elem_in_intersection(post_fixed_point_set(f), post_fixed_point_set(g), x) =
        (post_fixed_point_set(f).contains(x) and post_fixed_point_set(g).contains(x))
    post_fixed_point_set_contains_eq(f, x)
    post_fixed_point_set_contains_eq(g, x)
}

/// The common pre-fixed-point set is symmetric in its two maps.
theorem common_pre_fixed_point_set_comm[P: PartialOrder](f: P -> P, g: P -> P) {
    common_pre_fixed_point_set(f, g) = common_pre_fixed_point_set(g, f)
} by {
    forall(x: P) {
        common_pre_fixed_point_set_contains_eq(f, g, x)
        common_pre_fixed_point_set_contains_eq(g, f, x)
        common_pre_fixed_point_set(f, g).contains(x) =
            common_pre_fixed_point_set(g, f).contains(x)
    }
    set_ext(common_pre_fixed_point_set(f, g), common_pre_fixed_point_set(g, f))
}

/// The common post-fixed-point set is symmetric in its two maps.
theorem common_post_fixed_point_set_comm[P: PartialOrder](f: P -> P, g: P -> P) {
    common_post_fixed_point_set(f, g) = common_post_fixed_point_set(g, f)
} by {
    forall(x: P) {
        common_post_fixed_point_set_contains_eq(f, g, x)
        common_post_fixed_point_set_contains_eq(g, f, x)
        common_post_fixed_point_set(f, g).contains(x) =
            common_post_fixed_point_set(g, f).contains(x)
    }
    set_ext(common_post_fixed_point_set(f, g), common_post_fixed_point_set(g, f))
}

/// The common pre-fixed-point set is contained in the pre-fixed-point set of the first map.
theorem common_pre_fixed_point_set_subset_left[P: PartialOrder](f: P -> P, g: P -> P) {
    common_pre_fixed_point_set(f, g).subset(pre_fixed_point_set(f))
} by {
    forall(x: P) {
        if common_pre_fixed_point_set(f, g).contains(x) {
            common_pre_fixed_point_set_contains_eq(f, g, x)
            is_pre_fixed_point(f, x)
            pre_fixed_point_set_contains_eq(f, x)
            pre_fixed_point_set(f).contains(x)
        }
    }
}

/// The common pre-fixed-point set is contained in the pre-fixed-point set of the second map.
theorem common_pre_fixed_point_set_subset_right[P: PartialOrder](f: P -> P, g: P -> P) {
    common_pre_fixed_point_set(f, g).subset(pre_fixed_point_set(g))
} by {
    common_pre_fixed_point_set_comm(f, g)
    common_pre_fixed_point_set_subset_left(g, f)
}

/// The common post-fixed-point set is contained in the post-fixed-point set of the first map.
theorem common_post_fixed_point_set_subset_left[P: PartialOrder](f: P -> P, g: P -> P) {
    common_post_fixed_point_set(f, g).subset(post_fixed_point_set(f))
} by {
    forall(x: P) {
        if common_post_fixed_point_set(f, g).contains(x) {
            common_post_fixed_point_set_contains_eq(f, g, x)
            is_post_fixed_point(f, x)
            post_fixed_point_set_contains_eq(f, x)
            post_fixed_point_set(f).contains(x)
        }
    }
}

/// The common post-fixed-point set is contained in the post-fixed-point set of the second map.
theorem common_post_fixed_point_set_subset_right[P: PartialOrder](f: P -> P, g: P -> P) {
    common_post_fixed_point_set(f, g).subset(post_fixed_point_set(g))
} by {
    common_post_fixed_point_set_comm(f, g)
    common_post_fixed_point_set_subset_left(g, f)
}

/// The common pre-fixed points of a map with itself are exactly its pre-fixed points.
theorem common_pre_fixed_point_set_self[P: PartialOrder](f: P -> P) {
    common_pre_fixed_point_set(f, f) = pre_fixed_point_set(f)
} by {
    forall(x: P) {
        common_pre_fixed_point_set_contains_eq(f, f, x)
        pre_fixed_point_set_contains_eq(f, x)
        common_pre_fixed_point_set(f, f).contains(x) = pre_fixed_point_set(f).contains(x)
    }
    set_ext(common_pre_fixed_point_set(f, f), pre_fixed_point_set(f))
}

/// The common post-fixed points of a map with itself are exactly its post-fixed points.
theorem common_post_fixed_point_set_self[P: PartialOrder](f: P -> P) {
    common_post_fixed_point_set(f, f) = post_fixed_point_set(f)
} by {
    forall(x: P) {
        common_post_fixed_point_set_contains_eq(f, f, x)
        post_fixed_point_set_contains_eq(f, x)
        common_post_fixed_point_set(f, f).contains(x) = post_fixed_point_set(f).contains(x)
    }
    set_ext(common_post_fixed_point_set(f, f), post_fixed_point_set(f))
}

/// The common fixed-point set is the intersection of the common pre-fixed and common post-fixed sets.
theorem common_fixed_point_set_eq_common_pre_post_intersection[P: PartialOrder](f: P -> P, g: P -> P) {
    common_fixed_point_set(f, g) =
        common_pre_fixed_point_set(f, g).intersection(common_post_fixed_point_set(f, g))
} by {
    forall(x: P) {
        common_fixed_point_set_contains_eq(f, g, x)
        common_pre_fixed_point_set_contains_eq(f, g, x)
        common_post_fixed_point_set_contains_eq(f, g, x)
        if common_fixed_point_set(f, g).contains(x) {
            is_fixed_point(f, x)
            is_fixed_point(g, x)
            fixed_point_is_pre_fixed(f, x)
            fixed_point_is_post_fixed(f, x)
            fixed_point_is_pre_fixed(g, x)
            fixed_point_is_post_fixed(g, x)
            common_pre_fixed_point_set(f, g).contains(x)
            common_post_fixed_point_set(f, g).contains(x)
            elem_in_intersection(common_pre_fixed_point_set(f, g),
                common_post_fixed_point_set(f, g), x)
            common_pre_fixed_point_set(f, g).intersection(
                common_post_fixed_point_set(f, g)).contains(x)
        }
        if common_pre_fixed_point_set(f, g).intersection(
                common_post_fixed_point_set(f, g)).contains(x) {
            elem_in_intersection(common_pre_fixed_point_set(f, g),
                common_post_fixed_point_set(f, g), x)
            common_pre_fixed_point_set(f, g).contains(x)
            common_post_fixed_point_set(f, g).contains(x)
            is_pre_fixed_point(f, x)
            is_post_fixed_point(f, x)
            is_pre_fixed_point(g, x)
            is_post_fixed_point(g, x)
            fixed_point_of_pre_and_post(f, x)
            fixed_point_of_pre_and_post(g, x)
            common_fixed_point_set(f, g).contains(x)
        }
        common_fixed_point_set(f, g).contains(x) =
            common_pre_fixed_point_set(f, g).intersection(
                common_post_fixed_point_set(f, g)).contains(x)
    }
    set_ext(common_fixed_point_set(f, g),
        common_pre_fixed_point_set(f, g).intersection(common_post_fixed_point_set(f, g)))
}

/// The common fixed-point set is contained in the common pre-fixed-point set.
theorem common_fixed_point_set_subset_common_pre[P: PartialOrder](f: P -> P, g: P -> P) {
    common_fixed_point_set(f, g).subset(common_pre_fixed_point_set(f, g))
} by {
    common_fixed_point_set_eq_common_pre_post_intersection(f, g)
    sets_subset_intersection(common_pre_fixed_point_set(f, g),
        common_post_fixed_point_set(f, g))
}

/// The common fixed-point set is contained in the common post-fixed-point set.
theorem common_fixed_point_set_subset_common_post[P: PartialOrder](f: P -> P, g: P -> P) {
    common_fixed_point_set(f, g).subset(common_post_fixed_point_set(f, g))
} by {
    common_fixed_point_set_eq_common_pre_post_intersection(f, g)
    sets_subset_intersection(common_pre_fixed_point_set(f, g),
        common_post_fixed_point_set(f, g))
}

/// Pairing a map with the identity on the left leaves the common pre-fixed points unchanged.
theorem common_pre_fixed_point_set_identity_left[P: PartialOrder](g: P -> P) {
    common_pre_fixed_point_set(identity_fn[P], g) = pre_fixed_point_set(g)
} by {
    forall(x: P) {
        common_pre_fixed_point_set_contains_eq(identity_fn[P], g, x)
        identity_fn[P](x) = x
        is_pre_fixed_point(identity_fn[P], x)
        common_pre_fixed_point_set(identity_fn[P], g).contains(x) = is_pre_fixed_point(g, x)
        pre_fixed_point_set_contains_eq(g, x)
        common_pre_fixed_point_set(identity_fn[P], g).contains(x) =
            pre_fixed_point_set(g).contains(x)
    }
    set_ext(common_pre_fixed_point_set(identity_fn[P], g), pre_fixed_point_set(g))
}

/// Pairing a map with the identity on the right leaves the common pre-fixed points unchanged.
theorem common_pre_fixed_point_set_identity_right[P: PartialOrder](f: P -> P) {
    common_pre_fixed_point_set(f, identity_fn[P]) = pre_fixed_point_set(f)
} by {
    common_pre_fixed_point_set_comm(f, identity_fn[P])
    common_pre_fixed_point_set_identity_left(f)
}

/// Pairing a map with the identity on the left leaves the common post-fixed points unchanged.
theorem common_post_fixed_point_set_identity_left[P: PartialOrder](g: P -> P) {
    common_post_fixed_point_set(identity_fn[P], g) = post_fixed_point_set(g)
} by {
    forall(x: P) {
        common_post_fixed_point_set_contains_eq(identity_fn[P], g, x)
        identity_fn[P](x) = x
        is_post_fixed_point(identity_fn[P], x)
        common_post_fixed_point_set(identity_fn[P], g).contains(x) = is_post_fixed_point(g, x)
        post_fixed_point_set_contains_eq(g, x)
        common_post_fixed_point_set(identity_fn[P], g).contains(x) =
            post_fixed_point_set(g).contains(x)
    }
    set_ext(common_post_fixed_point_set(identity_fn[P], g), post_fixed_point_set(g))
}

/// Pairing a map with the identity on the right leaves the common post-fixed points unchanged.
theorem common_post_fixed_point_set_identity_right[P: PartialOrder](f: P -> P) {
    common_post_fixed_point_set(f, identity_fn[P]) = post_fixed_point_set(f)
} by {
    common_post_fixed_point_set_comm(f, identity_fn[P])
    common_post_fixed_point_set_identity_left(f)
}

/// The common fixed-point set is contained in the pre-fixed-point set of the first map.
theorem common_fixed_point_set_subset_pre_fixed_left[P: PartialOrder](f: P -> P, g: P -> P) {
    common_fixed_point_set(f, g).subset(pre_fixed_point_set(f))
} by {
    common_fixed_point_set_subset_common_pre(f, g)
    common_pre_fixed_point_set_subset_left(f, g)
    subset_trans(common_fixed_point_set(f, g),
        common_pre_fixed_point_set(f, g), pre_fixed_point_set(f))
}

/// The common fixed-point set is contained in the pre-fixed-point set of the second map.
theorem common_fixed_point_set_subset_pre_fixed_right[P: PartialOrder](f: P -> P, g: P -> P) {
    common_fixed_point_set(f, g).subset(pre_fixed_point_set(g))
} by {
    common_fixed_point_set_subset_common_pre(f, g)
    common_pre_fixed_point_set_subset_right(f, g)
    subset_trans(common_fixed_point_set(f, g),
        common_pre_fixed_point_set(f, g), pre_fixed_point_set(g))
}

/// The common fixed-point set is contained in the post-fixed-point set of the first map.
theorem common_fixed_point_set_subset_post_fixed_left[P: PartialOrder](f: P -> P, g: P -> P) {
    common_fixed_point_set(f, g).subset(post_fixed_point_set(f))
} by {
    common_fixed_point_set_subset_common_post(f, g)
    common_post_fixed_point_set_subset_left(f, g)
    subset_trans(common_fixed_point_set(f, g),
        common_post_fixed_point_set(f, g), post_fixed_point_set(f))
}

/// The common fixed-point set is contained in the post-fixed-point set of the second map.
theorem common_fixed_point_set_subset_post_fixed_right[P: PartialOrder](f: P -> P, g: P -> P) {
    common_fixed_point_set(f, g).subset(post_fixed_point_set(g))
} by {
    common_fixed_point_set_subset_common_post(f, g)
    common_post_fixed_point_set_subset_right(f, g)
    subset_trans(common_fixed_point_set(f, g),
        common_post_fixed_point_set(f, g), post_fixed_point_set(g))
}
