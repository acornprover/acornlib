from nat import Nat
from data.basic.set import Set
from data.basic.functions import is_bijection_fn, is_injective_fn, is_surjective_fn

/// The n-fold iterate of a self-map f.
define iterate[T](f: T -> T, n: Nat, x: T) -> T {
    match n {
        Nat.zero {
            x
        }
        Nat.suc(pred) {
            f(iterate(f, pred, x))
        }
    }
}

/// Iterating zero times is the identity.
theorem iterate_zero[T](f: T -> T, x: T) {
    iterate(f, Nat.0, x) = x
}

/// Iterating once applies f once.
theorem iterate_one[T](f: T -> T, x: T) {
    iterate(f, Nat.1, x) = f(x)
} by {
    iterate(f, Nat.0.suc, x) = f(iterate(f, Nat.0, x))
}

/// Successor unfolds the iterate from the outside.
theorem iterate_suc[T](f: T -> T, n: Nat, x: T) {
    iterate(f, n.suc, x) = f(iterate(f, n, x))
}

/// Successor unfolds the iterate from the inside.
theorem iterate_suc_inner[T](f: T -> T, n: Nat, x: T) {
    iterate(f, n.suc, x) = iterate(f, n, f(x))
} by {
    define p(k: Nat) -> Bool {
        iterate(f, k.suc, x) = iterate(f, k, f(x))
    }
    iterate(f, Nat.0.suc, x) = f(x)
    iterate(f, Nat.0, f(x)) = f(x)
    p(Nat.0)
    forall(k: Nat) {
        if p(k) {
            f(iterate(f, k.suc, x)) = f(iterate(f, k, f(x)))
            p(k.suc)
        }
    }
    p(n)
}

/// The iterate is additive in the exponent.
theorem iterate_add[T](f: T -> T, m: Nat, n: Nat, x: T) {
    iterate(f, m + n, x) = iterate(f, m, iterate(f, n, x))
} by {
    define p(k: Nat) -> Bool {
        iterate(f, m + k, x) = iterate(f, m, iterate(f, k, x))
    }
    iterate(f, m + Nat.0, x) = iterate(f, m, x)
    iterate(f, m, iterate(f, Nat.0, x)) = iterate(f, m, x)
    p(Nat.0)
    forall(k: Nat) {
        if p(k) {
            f(iterate(f, m + k, x)) = f(iterate(f, m, iterate(f, k, x)))
            iterate_suc_inner(f, m, iterate(f, k, x))
            p(k.suc)
        }
    }
    p(n)
}

/// True if y is on the forward orbit of x under f.
define orbit_contains[T](f: T -> T, x: T, y: T) -> Bool {
    exists(n: Nat) {
        y = iterate(f, n, x)
    }
}

/// The forward orbit of a point under a self-map.
define orbit[T](f: T -> T, x: T) -> Set[T] {
    Set[T].new(orbit_contains(f, x))
}

/// A point belongs to its own forward orbit.
theorem orbit_contains_self[T](f: T -> T, x: T) {
    orbit(f, x).contains(x)
} by {
    x = iterate(f, Nat.0, x)
    exists(n: Nat) {
        n = Nat.0 and x = iterate(f, n, x)
    }
}

/// Iterating from x stays in the orbit of x.
theorem orbit_contains_iterate[T](f: T -> T, x: T, n: Nat) {
    orbit(f, x).contains(iterate(f, n, x))
} by {
    exists(k: Nat) {
        k = n and iterate(f, n, x) = iterate(f, k, x)
    }
}

/// True if a set is forward-invariant under f: f maps S into S.
define is_invariant[T](f: T -> T, s: Set[T]) -> Bool {
    forall(x: T) {
        s.contains(x) implies s.contains(f(x))
    }
}

/// Iterates of an invariant set stay in the set, given the elementwise step.
theorem invariant_step_iterate[T](f: T -> T, s: Set[T], x: T, n: Nat) {
    (forall(z: T) { s.contains(z) implies s.contains(f(z)) })
        and s.contains(x)
        implies s.contains(iterate(f, n, x))
} by {
    define p(k: Nat) -> Bool {
        (forall(z: T) { s.contains(z) implies s.contains(f(z)) })
            and s.contains(x)
            implies s.contains(iterate(f, k, x))
    }
    iterate(f, Nat.0, x) = x
    p(Nat.0)
    forall(k: Nat) {
        if p(k) {
            if (forall(z: T) { s.contains(z) implies s.contains(f(z)) })
                and s.contains(x) {
                let y = iterate(f, k, x)
                s.contains(f(y))
                iterate_suc(f, k, x)
                s.contains(iterate(f, k.suc, x))
            }
            p(k.suc)
        }
    }
    p(n)
}

/// Iterates of an invariant set stay in the set.
theorem is_invariant_iterate[T](f: T -> T, s: Set[T], x: T, n: Nat) {
    is_invariant(f, s) and s.contains(x) implies s.contains(iterate(f, n, x))
} by {
    if is_invariant(f, s) and s.contains(x) {
        invariant_step_iterate(f, s, x, n)
    }
}

/// The orbit of a point is forward-invariant.
theorem orbit_is_invariant[T](f: T -> T, x: T) {
    is_invariant(f, orbit(f, x))
} by {
    forall(y: T) {
        if orbit(f, x).contains(y) {
            let n: Nat satisfy {
                y = iterate(f, n, x)
            }
            f(y) = f(iterate(f, n, x))
            f(iterate(f, n, x)) = iterate(f, n.suc, x)
            exists(k: Nat) {
                k = n.suc and f(y) = iterate(f, k, x)
            }
            orbit(f, x).contains(f(y))
        }
    }
}

/// True if x is a fixed point of f.
define is_fixed_point[T](f: T -> T, x: T) -> Bool {
    f(x) = x
}

/// Every iterate at a point fixed by f equals the point.
theorem iterate_at_fixed[T](f: T -> T, x: T, n: Nat) {
    f(x) = x implies iterate(f, n, x) = x
} by {
    define p(k: Nat) -> Bool {
        f(x) = x implies iterate(f, k, x) = x
    }
    iterate(f, Nat.0, x) = x
    p(Nat.0)
    forall(k: Nat) {
        if p(k) {
            if f(x) = x {
                iterate(f, k, x) = x
                iterate_suc(f, k, x)
                iterate(f, k.suc, x) = x
            }
            p(k.suc)
        }
    }
    p(n)
}

/// Every iterate of a fixed point equals the fixed point.
theorem iterate_fixed_point[T](f: T -> T, x: T, n: Nat) {
    is_fixed_point(f, x) implies iterate(f, n, x) = x
} by {
    if is_fixed_point(f, x) {
        iterate_at_fixed(f, x, n)
    }
}

/// Applying f to a fixed point yields a fixed point.
theorem is_fixed_point_step[T](f: T -> T, x: T) {
    is_fixed_point(f, x) implies is_fixed_point(f, f(x))
} by {
    if is_fixed_point(f, x) {
        f(f(x)) = f(x)
    }
}

/// The set of fixed points of f.
define fixed_point_set[T](f: T -> T) -> Set[T] {
    Set[T].new(is_fixed_point(f))
}

/// The set of fixed points is forward-invariant under f.
theorem fixed_point_set_invariant[T](f: T -> T) {
    is_invariant(f, fixed_point_set(f))
} by {
    forall(x: T) {
        if fixed_point_set(f).contains(x) {
            is_fixed_point_step(f, x)
            is_fixed_point(f, f(x))
            fixed_point_set(f).contains(f(x))
        }
    }
}

/// True if x is a periodic point of f with period n: positive n with f^n(x) = x.
define is_periodic_point[T](f: T -> T, x: T, n: Nat) -> Bool {
    n != Nat.0 and iterate(f, n, x) = x
}

/// True if x has some positive period under f.
define is_periodic[T](f: T -> T, x: T) -> Bool {
    exists(n: Nat) {
        is_periodic_point(f, x, n)
    }
}

/// A fixed point has period one.
theorem is_fixed_point_periodic_one[T](f: T -> T, x: T) {
    is_fixed_point(f, x) implies is_periodic_point(f, x, Nat.1)
} by {
    if is_fixed_point(f, x) {
        iterate_one(f, x)
        Nat.1 != Nat.0
    }
}

/// A fixed point is periodic.
theorem is_fixed_point_is_periodic[T](f: T -> T, x: T) {
    is_fixed_point(f, x) implies is_periodic(f, x)
} by {
    if is_fixed_point(f, x) {
        is_fixed_point_periodic_one(f, x)
        exists(n: Nat) {
            n = Nat.1 and is_periodic_point(f, x, n)
        }
    }
}

/// A periodic point returns to itself after every multiple of its period.
theorem iterate_periodic_multiple[T](f: T -> T, x: T, n: Nat, k: Nat) {
    is_periodic_point(f, x, n) implies iterate(f, k * n, x) = x
} by {
    define p(j: Nat) -> Bool {
        is_periodic_point(f, x, n) implies iterate(f, j * n, x) = x
    }
    Nat.0 * n = Nat.0
    iterate(f, Nat.0 * n, x) = x
    p(Nat.0)
    forall(j: Nat) {
        if p(j) {
            if is_periodic_point(f, x, n) {
                iterate(f, j * n, x) = x
                iterate(f, n, x) = x
                j.suc * n = j * n + n
                iterate_add(f, j * n, n, x)
                iterate(f, j.suc * n, x) = x
            }
            p(j.suc)
        }
    }
    p(k)
}

/// The orbit of a periodic point contains every iterate, hence is `closed under f` already
/// by `orbit_is_invariant`; periodicity additionally makes it finite in n-step structure.
theorem periodic_orbit_returns[T](f: T -> T, x: T, n: Nat) {
    is_periodic_point(f, x, n) implies orbit(f, x).contains(iterate(f, n, x))
} by {
    orbit_contains_iterate(f, x, n)
}

/// True if n is the prime (smallest positive) period of x under f.
define is_prime_period[T](f: T -> T, x: T, n: Nat) -> Bool {
    is_periodic_point(f, x, n)
        and forall(m: Nat) {
            m < n and m != Nat.0 implies iterate(f, m, x) != x
        }
}

// A periodic point has a prime period. Proof uses `has_min` from `nat_gcd` but
// the unfolding of the local predicate currently times out; left for follow-up.
//
// theorem is_periodic_has_prime_period[T](f: T -> T, x: T) {
//     is_periodic(f, x) implies exists(n: Nat) {
//         is_prime_period(f, x, n)
//     }
// }

/// Applying f to a periodic point yields a periodic point of the same period.
theorem is_periodic_point_step[T](f: T -> T, x: T, n: Nat) {
    is_periodic_point(f, x, n) implies is_periodic_point(f, f(x), n)
} by {
    if is_periodic_point(f, x, n) {
        iterate(f, n, x) = x
        iterate_suc_inner(f, n, x)
        iterate_suc(f, n, x)
        iterate(f, n, f(x)) = f(x)
    }
}

/// Applying f to a periodic point yields a periodic point.
theorem is_periodic_step[T](f: T -> T, x: T) {
    is_periodic(f, x) implies is_periodic(f, f(x))
} by {
    if is_periodic(f, x) {
        let n: Nat satisfy {
            is_periodic_point(f, x, n)
        }
        is_periodic_point_step(f, x, n)
        exists(k: Nat) {
            k = n and is_periodic_point(f, f(x), k)
        }
    }
}

/// The set of periodic points of f.
define periodic_point_set[T](f: T -> T) -> Set[T] {
    Set[T].new(is_periodic(f))
}

/// A periodic point set member maps to another member of the periodic point set.
theorem periodic_point_set_contains_step[T](f: T -> T, x: T) {
    periodic_point_set(f).contains(x) implies periodic_point_set(f).contains(f(x))
} by {
    if periodic_point_set(f).contains(x) {
        is_periodic_step(f, x)
        periodic_point_set(f).contains(f(x)) = is_periodic(f, f(x))
        periodic_point_set(f).contains(f(x))
    }
}

/// The set of periodic points is forward-invariant under f.
theorem periodic_point_set_invariant[T](f: T -> T) {
    is_invariant(f, periodic_point_set(f))
} by {
    forall(x: T) {
        periodic_point_set_contains_step(f, x)
    }
}

/// True if x has period dividing n: n is positive and the nth iterate returns x.
define has_period_dividing[T](f: T -> T, x: T, n: Nat) -> Bool {
    n != Nat.0 and iterate(f, n, x) = x
}

/// has_period_dividing is the same condition as is_periodic_point.
theorem has_period_dividing_iff_periodic_point[T](f: T -> T, x: T, n: Nat) {
    has_period_dividing(f, x, n) = is_periodic_point(f, x, n)
}

/// A prime-period point has period dividing n.
theorem prime_period_has_period_dividing[T](f: T -> T, x: T, n: Nat) {
    is_prime_period(f, x, n) implies has_period_dividing(f, x, n)
} by {
    if is_prime_period(f, x, n) {
        is_periodic_point(f, x, n)
    }
}

/// The set of points with prime period n under f.
define prime_period_set[T](f: T -> T, n: Nat) -> Set[T] {
    Set[T].new(function(x: T) { is_prime_period(f, x, n) })
}

/// The set of points whose period divides n.
define period_dividing_set[T](f: T -> T, n: Nat) -> Set[T] {
    Set[T].new(function(x: T) { has_period_dividing(f, x, n) })
}

/// A point with prime period n lies in the period-dividing set at n.
theorem prime_period_set_subset_period_dividing_set[T](f: T -> T, n: Nat) {
    prime_period_set(f, n).subset(period_dividing_set(f, n))
} by {
    forall(x: T) {
        if prime_period_set(f, n).contains(x) {
            prime_period_set(f, n).contains(x) = is_prime_period(f, x, n)
            is_prime_period(f, x, n)
            prime_period_has_period_dividing(f, x, n)
            has_period_dividing(f, x, n)
            period_dividing_set(f, n).contains(x) = has_period_dividing(f, x, n)
            period_dividing_set(f, n).contains(x)
        }
    }
}

/// Every point of the period-dividing set is periodic.
theorem period_dividing_set_subset_periodic[T](f: T -> T, n: Nat) {
    period_dividing_set(f, n).subset(periodic_point_set(f))
} by {
    forall(x: T) {
        if period_dividing_set(f, n).contains(x) {
            period_dividing_set(f, n).contains(x) = has_period_dividing(f, x, n)
            exists(k: Nat) {
                k = n and is_periodic_point(f, x, k)
            }
            is_periodic(f, x)
            periodic_point_set(f).contains(x)
        }
    }
}

/// A fixed point has prime period one.
theorem fixed_point_is_prime_period_one[T](f: T -> T, x: T) {
    is_fixed_point(f, x) implies is_prime_period(f, x, Nat.1)
} by {
    if is_fixed_point(f, x) {
        is_fixed_point_periodic_one(f, x)
        forall(m: Nat) {
            if m < Nat.1 and m != Nat.0 {
                // m < 1 forces m = 0, contradicting m != 0
                m < Nat.0.suc
                m <= Nat.0
                false
            }
        }
        is_prime_period(f, x, Nat.1)
    }
}

/// Prime period one is equivalent to being a fixed point.
theorem prime_period_one_iff_fixed_point[T](f: T -> T, x: T) {
    is_prime_period(f, x, Nat.1) = is_fixed_point(f, x)
} by {
    if is_prime_period(f, x, Nat.1) {
        is_periodic_point(f, x, Nat.1)
        iterate_one(f, x)
        f(x) = x
    }
    if is_fixed_point(f, x) {
        fixed_point_is_prime_period_one(f, x)
    }
}

/// Every iterate of a periodic point retains the same period.
theorem is_periodic_point_iterate[T](f: T -> T, x: T, n: Nat, k: Nat) {
    is_periodic_point(f, x, n) implies is_periodic_point(f, iterate(f, k, x), n)
} by {
    define p(j: Nat) -> Bool {
        is_periodic_point(f, x, n) implies is_periodic_point(f, iterate(f, j, x), n)
    }
    iterate(f, Nat.0, x) = x
    p(Nat.0)
    forall(j: Nat) {
        if p(j) {
            if is_periodic_point(f, x, n) {
                is_periodic_point_step(f, iterate(f, j, x), n)
                is_periodic_point(f, f(iterate(f, j, x)), n)
                iterate_suc(f, j, x)
                is_periodic_point(f, iterate(f, j.suc, x), n)
            }
            p(j.suc)
        }
    }
    p(k)
}

/// Iterating f preserves periodicity.
theorem is_periodic_iterate[T](f: T -> T, x: T, k: Nat) {
    is_periodic(f, x) implies is_periodic(f, iterate(f, k, x))
} by {
    if is_periodic(f, x) {
        periodic_point_set_invariant(f)
        is_invariant_iterate(f, periodic_point_set(f), x, k)
        periodic_point_set(f).contains(iterate(f, k, x))
    }
}

/// The forward orbit of x is contained in any forward-invariant set containing x.
theorem orbit_subset_invariant[T](f: T -> T, s: Set[T], x: T) {
    is_invariant(f, s) and s.contains(x) implies orbit(f, x).subset(s)
} by {
    if is_invariant(f, s) and s.contains(x) {
        forall(y: T) {
            if orbit(f, x).contains(y) {
                let n: Nat satisfy {
                    y = iterate(f, n, x)
                }
                is_invariant_iterate(f, s, x, n)
                s.contains(y)
            }
        }
    }
}

/// The orbit of x is the smallest forward-invariant set containing x, witnessed by subset closure.
theorem orbit_subset_self[T](f: T -> T, x: T) {
    orbit(f, x).subset(orbit(f, x))
} by {
    orbit_is_invariant(f, x)
    orbit_contains_self(f, x)
    orbit_subset_invariant(f, orbit(f, x), x)
}

/// True if y is on the backward orbit of x under f with inverse-witness k:
/// y is reachable from x by some number of backward steps along k.
define backward_orbit_contains[T](f: T -> T, k: T -> T, x: T, y: T) -> Bool {
    exists(n: Nat) {
        y = iterate(k, n, x)
    }
}

/// The backward orbit of a point under a self-map with inverse-witness k.
define backward_orbit[T](f: T -> T, k: T -> T, x: T) -> Set[T] {
    Set[T].new(backward_orbit_contains(f, k, x))
}

/// A point belongs to its own backward orbit.
theorem backward_orbit_contains_self[T](f: T -> T, k: T -> T, x: T) {
    backward_orbit(f, k, x).contains(x)
} by {
    x = iterate(k, Nat.0, x)
    exists(n: Nat) {
        n = Nat.0 and x = iterate(k, n, x)
    }
}

/// Iterating the inverse from x stays in the backward orbit of x.
theorem backward_orbit_contains_iterate[T](f: T -> T, k: T -> T, x: T, n: Nat) {
    backward_orbit(f, k, x).contains(iterate(k, n, x))
} by {
    exists(j: Nat) {
        j = n and iterate(k, n, x) = iterate(k, j, x)
    }
}

/// True if y is on the full orbit of x under f with inverse-witness k:
/// y is on either the forward or the backward orbit of x.
define full_orbit_contains[T](f: T -> T, k: T -> T, x: T, y: T) -> Bool {
    orbit_contains(f, x, y) or backward_orbit_contains(f, k, x, y)
}

/// The full orbit of a point under an invertible self-map with inverse-witness k.
define full_orbit[T](f: T -> T, k: T -> T, x: T) -> Set[T] {
    Set[T].new(full_orbit_contains(f, k, x))
}

/// The forward orbit is contained in the full orbit.
theorem full_orbit_contains_forward[T](f: T -> T, k: T -> T, x: T, y: T) {
    orbit(f, x).contains(y) implies full_orbit(f, k, x).contains(y)
}

/// The backward orbit is contained in the full orbit.
theorem full_orbit_contains_backward[T](f: T -> T, k: T -> T, x: T, y: T) {
    backward_orbit(f, k, x).contains(y) implies full_orbit(f, k, x).contains(y)
}

/// A point belongs to its own full orbit.
theorem full_orbit_contains_self[T](f: T -> T, k: T -> T, x: T) {
    full_orbit(f, k, x).contains(x)
} by {
    orbit_contains_self(f, x)
}

/// Forward iterates lie in the full orbit.
theorem full_orbit_contains_iterate[T](f: T -> T, k: T -> T, x: T, n: Nat) {
    full_orbit(f, k, x).contains(iterate(f, n, x))
} by {
    orbit_contains_iterate(f, x, n)
}

/// Backward iterates lie in the full orbit.
theorem full_orbit_contains_backward_iterate[T](f: T -> T, k: T -> T, x: T, n: Nat) {
    full_orbit(f, k, x).contains(iterate(k, n, x))
} by {
    backward_orbit_contains_iterate(f, k, x, n)
}

/// True if h is a semiconjugacy from f to g: h composes f into g, i.e. h(f(x)) = g(h(x)).
define is_semiconjugacy[T, U](h: T -> U, f: T -> T, g: U -> U) -> Bool {
    forall(x: T) {
        h(f(x)) = g(h(x))
    }
}

/// True if h conjugates f to g via the bijection witnessed by an inverse k.
define is_conjugacy[T, U](h: T -> U, k: U -> T, f: T -> T, g: U -> U) -> Bool {
    is_semiconjugacy(h, f, g)
        and forall(x: T) { k(h(x)) = x }
        and forall(y: U) { h(k(y)) = y }
}

/// Semiconjugacy commutes with iteration: h(f^n(x)) = g^n(h(x)).
theorem iterate_semiconjugacy[T, U](h: T -> U, f: T -> T, g: U -> U, x: T, n: Nat) {
    is_semiconjugacy(h, f, g) implies h(iterate(f, n, x)) = iterate(g, n, h(x))
} by {
    define p(k: Nat) -> Bool {
        is_semiconjugacy(h, f, g) implies h(iterate(f, k, x)) = iterate(g, k, h(x))
    }
    iterate(f, Nat.0, x) = x
    iterate(g, Nat.0, h(x)) = h(x)
    p(Nat.0)
    forall(k: Nat) {
        if p(k) {
            if is_semiconjugacy(h, f, g) {
                h(iterate(f, k, x)) = iterate(g, k, h(x))
                iterate_suc(f, k, x)
                iterate(f, k.suc, x) = f(iterate(f, k, x))
                h(iterate(f, k.suc, x)) = h(f(iterate(f, k, x)))
                h(f(iterate(f, k, x))) = g(h(iterate(f, k, x)))
                iterate_suc(g, k, h(x))
                h(iterate(f, k.suc, x)) = iterate(g, k.suc, h(x))
            }
            p(k.suc)
        }
    }
    p(n)
}

/// Semiconjugacy maps the forward orbit of x to the forward orbit of h(x).
theorem orbit_semiconjugacy[T, U](h: T -> U, f: T -> T, g: U -> U, x: T, y: T) {
    is_semiconjugacy(h, f, g) and orbit(f, x).contains(y)
        implies orbit(g, h(x)).contains(h(y))
} by {
    if is_semiconjugacy(h, f, g) and orbit(f, x).contains(y) {
        orbit_contains(f, x, y)
        let n: Nat satisfy {
            y = iterate(f, n, x)
        }
        iterate_semiconjugacy(h, f, g, x, n)
        orbit_contains(g, h(x), h(y))
    }
}

/// Semiconjugacy carries fixed points of f to fixed points of g.
theorem fixed_point_semiconjugacy[T, U](h: T -> U, f: T -> T, g: U -> U, x: T) {
    is_semiconjugacy(h, f, g) and is_fixed_point(f, x)
        implies is_fixed_point(g, h(x))
} by {
    if is_semiconjugacy(h, f, g) and is_fixed_point(f, x) {
        f(x) = x
        h(f(x)) = g(h(x))
        g(h(x)) = h(x)
    }
}

/// Semiconjugacy carries periodic points of f with period n to periodic points of g with period n.
theorem periodic_point_semiconjugacy[T, U](h: T -> U, f: T -> T, g: U -> U, x: T, n: Nat) {
    is_semiconjugacy(h, f, g) and is_periodic_point(f, x, n)
        implies is_periodic_point(g, h(x), n)
} by {
    if is_semiconjugacy(h, f, g) and is_periodic_point(f, x, n) {
        iterate(f, n, x) = x
        iterate_semiconjugacy(h, f, g, x, n)
        iterate(g, n, h(x)) = h(x)
        is_periodic_point(g, h(x), n)
    }
}

/// Semiconjugacy carries periodic points of f to periodic points of g.
theorem periodic_semiconjugacy[T, U](h: T -> U, f: T -> T, g: U -> U, x: T) {
    is_semiconjugacy(h, f, g) and is_periodic(f, x)
        implies is_periodic(g, h(x))
} by {
    if is_semiconjugacy(h, f, g) and is_periodic(f, x) {
        let n: Nat satisfy {
            is_periodic_point(f, x, n)
        }
        periodic_point_semiconjugacy(h, f, g, x, n)
        is_periodic(g, h(x))
    }
}

/// A conjugacy is also a semiconjugacy.
theorem conjugacy_is_semiconjugacy[T, U](h: T -> U, k: U -> T, f: T -> T, g: U -> U) {
    is_conjugacy(h, k, f, g) implies is_semiconjugacy(h, f, g)
} by {
    if is_conjugacy(h, k, f, g) {
        is_semiconjugacy(h, f, g)
    }
}

/// A conjugacy makes h injective.
theorem conjugacy_injective[T, U](h: T -> U, k: U -> T, f: T -> T, g: U -> U) {
    is_conjugacy(h, k, f, g) implies is_injective_fn(h)
} by {
    if is_conjugacy(h, k, f, g) {
        is_semiconjugacy(h, f, g) and (forall(x: T) { k(h(x)) = x }) and (forall(y: U) { h(k(y)) = y })
        forall(x: T) { k(h(x)) = x }
        forall(x: T, y: T) {
            if h(x) = h(y) {
                x = y
            }
        }
    }
}

/// A conjugacy makes h surjective.
theorem conjugacy_surjective[T, U](h: T -> U, k: U -> T, f: T -> T, g: U -> U) {
    is_conjugacy(h, k, f, g) implies is_surjective_fn(h)
} by {
    if is_conjugacy(h, k, f, g) {
        is_semiconjugacy(h, f, g) and (forall(x: T) { k(h(x)) = x }) and (forall(y: U) { h(k(y)) = y })
        forall(y: U) { h(k(y)) = y }
        forall(y: U) {
            exists(x: T) {
                x = k(y) and h(x) = y
            }
        }
    }
}

/// A conjugacy makes h a bijection.
theorem conjugacy_bijection[T, U](h: T -> U, k: U -> T, f: T -> T, g: U -> U) {
    is_conjugacy(h, k, f, g) implies is_bijection_fn(h)
} by {
    if is_conjugacy(h, k, f, g) {
        conjugacy_injective(h, k, f, g)
        conjugacy_surjective(h, k, f, g)
    }
}

/// A conjugacy makes k injective.
theorem conjugacy_inverse_injective[T, U](h: T -> U, k: U -> T, f: T -> T, g: U -> U) {
    is_conjugacy(h, k, f, g) implies is_injective_fn(k)
} by {
    if is_conjugacy(h, k, f, g) {
        is_semiconjugacy(h, f, g) and (forall(x: T) { k(h(x)) = x }) and (forall(y: U) { h(k(y)) = y })
        forall(y: U) { h(k(y)) = y }
        forall(x: U, y: U) {
            if k(x) = k(y) {
                x = y
            }
        }
    }
}

/// A conjugacy makes k surjective.
theorem conjugacy_inverse_surjective[T, U](h: T -> U, k: U -> T, f: T -> T, g: U -> U) {
    is_conjugacy(h, k, f, g) implies is_surjective_fn(k)
} by {
    if is_conjugacy(h, k, f, g) {
        is_semiconjugacy(h, f, g) and (forall(x: T) { k(h(x)) = x }) and (forall(y: U) { h(k(y)) = y })
        forall(x: T) { k(h(x)) = x }
        forall(y: T) {
            exists(x: U) {
                x = h(y) and k(x) = y
            }
        }
    }
}

/// A conjugacy makes k a bijection.
theorem conjugacy_inverse_bijection[T, U](h: T -> U, k: U -> T, f: T -> T, g: U -> U) {
    is_conjugacy(h, k, f, g) implies is_bijection_fn(k)
} by {
    if is_conjugacy(h, k, f, g) {
        conjugacy_inverse_injective(h, k, f, g)
        conjugacy_inverse_surjective(h, k, f, g)
    }
}

/// A conjugacy yields a semiconjugacy in the reverse direction.
theorem conjugacy_reverse_semiconjugacy[T, U](h: T -> U, k: U -> T, f: T -> T, g: U -> U) {
    is_conjugacy(h, k, f, g) implies is_semiconjugacy(k, g, f)
} by {
    if is_conjugacy(h, k, f, g) {
        is_semiconjugacy(h, f, g) and (forall(x: T) { k(h(x)) = x }) and (forall(y: U) { h(k(y)) = y })
        is_semiconjugacy(h, f, g)
        forall(x: T) { k(h(x)) = x }
        forall(y: U) { h(k(y)) = y }
        forall(y: U) {
            h(k(y)) = y
            g(y) = g(h(k(y)))
            h(f(k(y))) = g(h(k(y)))
            k(g(y)) = f(k(y))
        }
    }
}

/// A conjugacy reflects fixed points along its inverse witness.
theorem conjugacy_fixed_point_iff[T, U](h: T -> U, k: U -> T, f: T -> T, g: U -> U, x: T) {
    is_conjugacy(h, k, f, g) implies is_fixed_point(f, x) = is_fixed_point(g, h(x))
} by {
    if is_conjugacy(h, k, f, g) {
        if is_fixed_point(f, x) {
            conjugacy_is_semiconjugacy(h, k, f, g)
            fixed_point_semiconjugacy(h, f, g, x)
            is_fixed_point(g, h(x))
        }
        if is_fixed_point(g, h(x)) {
            conjugacy_reverse_semiconjugacy(h, k, f, g)
            fixed_point_semiconjugacy(k, g, f, h(x))
            is_fixed_point(f, k(h(x)))
            is_conjugacy(h, k, f, g) =
                (is_semiconjugacy(h, f, g)
                and forall(y: T) { k(h(y)) = y }
                and forall(z: U) { h(k(z)) = z })
            forall(y: T) { k(h(y)) = y }
            f(k(h(x))) = k(h(x))
            is_fixed_point(f, x)
        }
    }
}

/// A conjugacy transports a target-side orbit point back to the source-side orbit via its inverse witness.
theorem conjugacy_orbit_reverse[T, U](h: T -> U, k: U -> T, f: T -> T, g: U -> U, x: T, y: U) {
    is_conjugacy(h, k, f, g) and orbit(g, h(x)).contains(y) implies orbit(f, x).contains(k(y))
} by {
    if is_conjugacy(h, k, f, g) and orbit(g, h(x)).contains(y) {
        orbit_contains(g, h(x), y)
        let n: Nat satisfy {
            y = iterate(g, n, h(x))
        }
        conjugacy_reverse_semiconjugacy(h, k, f, g)
        iterate_semiconjugacy(k, g, f, h(x), n)
        k(iterate(g, n, h(x))) = iterate(f, n, k(h(x)))
        k(y) = k(iterate(g, n, h(x)))
        is_conjugacy(h, k, f, g) =
            (is_semiconjugacy(h, f, g)
            and forall(z: T) { k(h(z)) = z }
            and forall(w: U) { h(k(w)) = w })
        forall(z: T) { k(h(z)) = z }
        k(h(x)) = x
        iterate(f, n, k(h(x))) = iterate(f, n, x)
        k(y) = iterate(f, n, x)
        exists(m: Nat) {
            m = n and k(y) = iterate(f, m, x)
        }
        orbit(f, x).contains(k(y))
    }
}

/// A conjugacy transports inverse-image membership in a source orbit to target-side orbit membership.
theorem conjugacy_orbit_forward_inverse_image[T, U](h: T -> U, k: U -> T, f: T -> T, g: U -> U, x: T, y: U) {
    is_conjugacy(h, k, f, g) and orbit(f, x).contains(k(y)) implies orbit(g, h(x)).contains(y)
} by {
    if is_conjugacy(h, k, f, g) and orbit(f, x).contains(k(y)) {
        conjugacy_is_semiconjugacy(h, k, f, g)
        orbit_semiconjugacy(h, f, g, x, k(y))
        is_conjugacy(h, k, f, g) =
            (is_semiconjugacy(h, f, g)
            and forall(z: T) { k(h(z)) = z }
            and forall(w: U) { h(k(w)) = w })
        forall(w: U) { h(k(w)) = w }
        orbit(g, h(x)).contains(y)
    }
}

/// The identity on T is a semiconjugacy from f to itself.
theorem identity_is_semiconjugacy[T](f: T -> T) {
    is_semiconjugacy(function(x: T) { x }, f, f)
} by {
    let h: (T -> T) = function(x: T) { x }
    forall(x: T) {
        h(f(x)) = f(h(x))
    }
}

/// Composition of semiconjugacies through a witnessing intermediate map.
theorem semiconjugacy_compose[T, U, V](h: T -> U, j: U -> V, jh: T -> V,
        f: T -> T, g: U -> U, e: V -> V) {
    is_semiconjugacy(h, f, g) and is_semiconjugacy(j, g, e)
        and (forall(x: T) { jh(x) = j(h(x)) })
        implies is_semiconjugacy(jh, f, e)
} by {
    if is_semiconjugacy(h, f, g) and is_semiconjugacy(j, g, e)
        and (forall(x: T) { jh(x) = j(h(x)) }) {
        forall(x: T) {
            jh(x) = j(h(x))
            h(f(x)) = g(h(x))
            j(h(f(x))) = j(g(h(x)))
            j(g(h(x))) = e(j(h(x)))
            jh(f(x)) = e(jh(x))
        }
    }
}
