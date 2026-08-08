from data.basic.functions import compose, identity_fn, function_extensionality,
    is_left_inverse_fn, is_right_inverse_fn
from analysis.dynamical.dynamical_systems import is_semiconjugacy, is_conjugacy,
    conjugacy_is_semiconjugacy, conjugacy_reverse_semiconjugacy,
    semiconjugacy_compose

/// The identity on a dynamical system is a conjugacy from the system to itself.
theorem identity_is_conjugacy[T](f: T -> T) {
    is_conjugacy(identity_fn[T], identity_fn[T], f, f)
} by {
    forall(x: T) {
        identity_fn[T](f(x)) = f(x)
        f(identity_fn[T](x)) = f(x)
        identity_fn[T](f(x)) = f(identity_fn[T](x))
    }
    is_semiconjugacy(identity_fn[T], f, f)
    forall(x: T) {
        identity_fn[T](identity_fn[T](x)) = x
    }
    forall(y: T) {
        identity_fn[T](identity_fn[T](y)) = y
    }
    is_conjugacy(identity_fn[T], identity_fn[T], f, f)
}

/// The inverse witness of a conjugacy is itself a conjugacy in the reverse direction.
theorem conjugacy_inverse_is_conjugacy[T, U](h: T -> U, k: U -> T, f: T -> T, g: U -> U) {
    is_conjugacy(h, k, f, g) implies is_conjugacy(k, h, g, f)
} by {
    if is_conjugacy(h, k, f, g) {
        conjugacy_reverse_semiconjugacy(h, k, f, g)
        is_semiconjugacy(k, g, f)
        is_conjugacy(h, k, f, g) =
            (is_semiconjugacy(h, f, g)
                and forall(x: T) { k(h(x)) = x }
                and forall(y: U) { h(k(y)) = y })
        forall(x: T) { k(h(x)) = x }
        forall(y: U) { h(k(y)) = y }
        is_conjugacy(k, h, g, f)
    }
}

/// Conjugacies compose.
theorem conjugacy_compose[T, U, V](h: T -> U, k: U -> T, j: U -> V, l: V -> U,
        f: T -> T, g: U -> U, e: V -> V) {
    is_conjugacy(h, k, f, g) and is_conjugacy(j, l, g, e) implies
        is_conjugacy(compose(j, h), compose(k, l), f, e)
} by {
    if is_conjugacy(h, k, f, g) and is_conjugacy(j, l, g, e) {
        let jh: T -> V = function(x: T) { j(h(x)) }
        let kl: V -> T = function(z: V) { k(l(z)) }
        conjugacy_is_semiconjugacy(h, k, f, g)
        is_semiconjugacy(h, f, g)
        conjugacy_is_semiconjugacy(j, l, g, e)
        is_semiconjugacy(j, g, e)
        forall(x: T) {
            jh(x) = j(h(x))
        }
        semiconjugacy_compose(h, j, jh, f, g, e)
        is_semiconjugacy(jh, f, e)
        is_conjugacy(h, k, f, g) =
            (is_semiconjugacy(h, f, g)
                and forall(x: T) { k(h(x)) = x }
                and forall(y: U) { h(k(y)) = y })
        forall(x: T) { k(h(x)) = x }
        forall(y: U) { h(k(y)) = y }
        is_conjugacy(j, l, g, e) =
            (is_semiconjugacy(j, g, e)
                and forall(y: U) { l(j(y)) = y }
                and forall(z: V) { j(l(z)) = z })
        forall(y: U) { l(j(y)) = y }
        forall(z: V) { j(l(z)) = z }
        forall(x: T) {
            kl(jh(x)) = k(l(jh(x)))
            jh(x) = j(h(x))
            k(l(jh(x))) = k(l(j(h(x))))
            l(j(h(x))) = h(x)
            k(l(j(h(x)))) = k(h(x))
            k(h(x)) = x
            kl(jh(x)) = x
        }
        forall(z: V) {
            jh(kl(z)) = j(h(kl(z)))
            kl(z) = k(l(z))
            j(h(kl(z))) = j(h(k(l(z))))
            h(k(l(z))) = l(z)
            j(h(k(l(z)))) = j(l(z))
            j(l(z)) = z
            jh(kl(z)) = z
        }
        is_left_inverse_fn(jh, kl)
        is_right_inverse_fn(jh, kl)
        is_conjugacy(jh, kl, f, e)
        forall(x: T) {
            compose(j, h, x) = j(h(x))
            jh(x) = j(h(x))
            compose(j, h, x) = jh(x)
        }
        function_extensionality(compose(j, h), jh)
        compose(j, h) = jh
        forall(z: V) {
            compose(k, l, z) = k(l(z))
            kl(z) = k(l(z))
            compose(k, l, z) = kl(z)
        }
        function_extensionality(compose(k, l), kl)
        compose(k, l) = kl
        is_conjugacy(compose(j, h), compose(k, l), f, e)
    }
}
