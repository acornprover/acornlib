/// Fourier series: convergence-oriented developments on top of the
/// trigonometric-system foundations of `analysis/fourier_basics` and /// `analysis/fourier_foundations`.

///
/// What exists upstream:
///   - the Fourier coefficients a_n, b_n of a function f on [0, 2*pi]
///     (`fourier_cos_coeff`, `fourier_sin_coeff` in `fourier_basics`), with
///     the concrete integral computations for the harmonics x.sin, x.cos, ///     (2x).sin and the orthogonality of the first few of them, ///   - the Dirichlet-kernel summation identity


///     2 (x/2).sin * (1/2 + sum_{k=1}^{n} kx.cos) = ((n + 1/2) x).sin
///     (`dirichlet_kernel_identity` in `fourier_foundations`).
///
/// This module adds:
///   - the missing linear-substitution instances for the trig harmonics: the
///     integral of kx.sin and kx.cos over one full period [0, 2*pi] is zero
///     for every k >= 1, and the norms integral(kx.sin^2) = integral(kx.cos^2)
///     = pi for every k >= 1 (the general orthogonality and normalization of
///     the trigonometric system), obtained from the fundamental theorem with
///     explicit antiderivatives, ///   - the concrete Fourier coefficients of the harmonics themselves

///     (task 1(a): a_0, a_1, b_1 for Real.sin and Real.cos, and the n = 2 instances), ///     and the n-th partial sum S_n of the Fourier series, ///   - the real Dirichlet kernel D_n(t) = 1 + 2 sum_{k=1}^{n} kt.cos: its


///     closed form D_n(t) = ((n + 1/2)t).sin / (t/2).sin away from the zeros of
///     (t/2).sin (an algebraic consequence of the proved summation identity), ///     and its integral property (1/2*pi) integral(D_n, 0, 2*pi) = 1, ///   - an integral-additivity theorem for pairs of functions with known


///     antiderivatives (the general Darboux additivity is future work in
///     `real/integral.ac`), from which the linearity of the Fourier
///     coefficients follows, and ///   - Parseval's identity in the basic cases: the mean square of a single

///     harmonic with coefficient c is c^2 / 2.
///
/// Not attempted here (documented at the bottom): the general convolution
/// identity S_n(f) = f * D_n (needs the substitution x - t in the Darboux
/// integral, which does not exist in the library), the general orthogonality
/// for products of distinct harmonics (needs the same substitution), and the
/// Riemann-Lebesgue lemma (needs a general oscillation argument).

from nat import Nat, from_nat, from_nat_zero, from_nat_add, from_nat_mul, zero_or_suc, alt_induction, lt_imp_lte_suc, suc_ne, lt_not_ref, lt_add_suc, lt_suc

from real import Real, pi, pi_pos, two, two_positive, two_nonzero, one_half_plus_one_half, partial_suc, sin_zero, cos_zero, sin_two_pi_zero, cos_two_pi_one, sin_add, cos_add, sin_neg, cos_neg, sin_sq_add_cos_sq, sin_continuous, cos_continuous, sin_lipschitz, cos_lipschitz, sin_abs_le_one, cos_abs_le_one, integral, is_integrable, interval_contains, interval_contains_left,
 interval_contains_right, ftc2_general, fn_integrable_gen, is_derivative_fn,
 derivative_fn_identity, derivative_fn_constant, derivative_fn_neg,
 derivative_fn_add, derivative_fn_const_mul, derivative_fn_mul,
 derivative_fn_square, sin_is_derivative_fn, cos_derivative_is_neg_sin,
 continuous, continuous_pointwise_neg, continuous_pointwise_add,
 continuous_pointwise_mul, constant_function_is_continuous,
 identity_function_is_continuous, continuous_const_mul_left, const_mul_left,
 abs_of_nonneg, neg_lte_flip, add_comm, add_assoc, neg_distrib, abs_neg,
 neg_neg, lte_abs, abs_gte_zero, gt_zero_imp_pos, pos_gt_zero, sub_cancels,
 lte_self, neg_zero, pos_imp_eq_abs, mul_distrib_left, mul_distrib_right,
 mul_sub_distrib_left, mul_sub_distrib_right, mul_abs, triangle_ineq,
 mul_inverse, mul_div, mul_div_cancel, div_mul_cancel_left,
 div_mul_cancel_right, sub_zero_imp_eq, from_nat_suc_pos_real,
 zero_is_different_than_one, one_half_positive, add_neg_eq_zero,
 add_zero_right, add_zero_left, mul_nonneg, real_mul_comm, mul_assoc,
 sub_moves_sides, mul_neg_one_left, mul_neg_one_right, mul_neg_right, mul_div_left, from_nat_is_from_rat, lte_add_right




from order import lt_imp_lte, lt_imp_ne, lte_trans, lt_trans
from algebra.add_ordered_group import add_le_add, add_le_add_right, add_lt_add_right

from ordered_field import mul_le_mul_of_nonneg_right, mul_le_mul_of_nonneg_left
from algebra.field.field import mul_not_zero
from data.basic.function_algebra import pointwise_add, pointwise_neg, pointwise_mul, pointwise_mul_apply, pointwise_add_apply, pointwise_neg_apply

from data.basic.functions import compose, identity_fn, function_extensionality
from list import partial, partial_add, partial_scalar_mul, partial_pointwise_eq, partial_zero
from algebra.semigroup import mul_fn
from algebra.add_semigroup import add_fn
from analysis.fourier_basics import two_pi, two_pi_pos, two_pi_nonneg, sin_nt, cos_nt, fourier_cos_coeff, fourier_sin_coeff, fourier_cos_integrand, fourier_sin_integrand, fn_sincos, fn_sin_sq, fn_cos_sq, fn_sin_sin2, integral_sin_mul_cos_zero, integral_sin_sq_zero_two_pi, integral_cos_sq_zero_two_pi, integral_sin_sin2_zero, sin_double,
 sub_add_cancel, add_sub_sub_cancel, add_sub_add_cancel, add_self_two, half_twice,
 one_nonneg, abs_le_one_imp_upper, abs_le_one_imp_lower, abs_le_imp_lower,
 abs_le_imp_upper, abs_mul_le_one, sin_add_abs_le_two, cos_add_abs_le_two, one_add_two_is_three,
 half_pythag_then_sin, half_pythag_then_cos, fourier_sin_coeff_identity_one




from analysis.fourier_foundations import sincos, sincos_lipschitz_m2, cos_harmonic, cos_harmonic_sum, half_angle_harmonic, dirichlet_kernel_identity, sin_sq_times_two, cos_sq_times_two, product_to_sum_sin_cos, product_to_sum_sin_sin, product_to_sum_cos_cos, integral_sin_zero_two_pi, integral_cos_zero_two_pi




numerals Real
numerals Nat

// ---------------------------------------------------------------------------
// Special values of the harmonics at the endpoints of the period
// ---------------------------------------------------------------------------
//
// (k * 2*pi).sin = 0 and (k * 2*pi).cos = 1 for every natural k, by induction
// from the addition laws and (2*pi).sin = 0, (2*pi).cos = 1.  These are the
// periodicity facts behind every integral computation over [0, 2*pi].

/// (0 * x).sin = 0 for every x.
theorem sin_nt_zero_harmonic(x: Real) {
    sin_nt(Nat.0, x) = Real.0
} by {
    sin_nt(Nat.0, x) = (from_nat[Real](Nat.0) * x).sin
    from_nat_zero[Real]
    from_nat[Real](Nat.0) = Real.0
    from_nat[Real](Nat.0) * x = Real.0 * x
    Real.0 * x = Real.0
    (Real.0 * x).sin = (Real.0).sin
    sin_zero
    (Real.0).sin = Real.0
    sin_nt(Nat.0, x) = Real.0
}

/// (0 * x).cos = 1 for every x.
theorem cos_nt_zero_harmonic(x: Real) {
    cos_nt(Nat.0, x) = Real.1
} by {
    cos_nt(Nat.0, x) = (from_nat[Real](Nat.0) * x).cos
    from_nat_zero[Real]
    from_nat[Real](Nat.0) = Real.0
    from_nat[Real](Nat.0) * x = Real.0 * x
    Real.0 * x = Real.0
    (Real.0 * x).cos = (Real.0).cos
    cos_zero
    (Real.0).cos = Real.1
    cos_nt(Nat.0, x) = Real.1
}

/// (k * 0).sin = 0 for every k.
theorem sin_nt_at_zero(k: Nat) {
    sin_nt(k, Real.0) = Real.0
} by {
    sin_nt(k, Real.0) = (from_nat[Real](k) * Real.0).sin
    from_nat[Real](k) * Real.0 = Real.0
    (from_nat[Real](k) * Real.0).sin = (Real.0).sin
    sin_zero
    (Real.0).sin = Real.0
    sin_nt(k, Real.0) = Real.0
}

/// (k * 0).cos = 1 for every k.
theorem cos_nt_at_zero(k: Nat) {
    cos_nt(k, Real.0) = Real.1
} by {
    cos_nt(k, Real.0) = (from_nat[Real](k) * Real.0).cos
    from_nat[Real](k) * Real.0 = Real.0
    (from_nat[Real](k) * Real.0).cos = (Real.0).cos
    cos_zero
    (Real.0).cos = Real.1
    cos_nt(k, Real.0) = Real.1
}

/// The successor harmonic at the period: ((k + 1) * 2*pi).sin in terms of
/// (k * 2*pi).sin and (k * 2*pi).cos.
theorem sin_nt_suc_at_two_pi(k: Nat) {
    sin_nt(k.suc, two_pi) = sin_nt(k, two_pi) * two_pi.cos + cos_nt(k, two_pi) * two_pi.sin
} by {
    sin_nt(k.suc, two_pi) = (from_nat[Real](k.suc) * two_pi).sin
    from_nat_add[Real](k, Nat.1)
    from_nat[Real](k + Nat.1) = from_nat[Real](k) + from_nat[Real](Nat.1)
    k + Nat.1 = k.suc
    from_nat[Real](k.suc) = from_nat[Real](k) + from_nat[Real](Nat.1)
    from_nat[Real](Nat.1) = Real.1
    from_nat[Real](k.suc) = from_nat[Real](k) + Real.1
    mul_distrib_left(from_nat[Real](k), Real.1, two_pi)
    (from_nat[Real](k) + Real.1) * two_pi = from_nat[Real](k) * two_pi + Real.1 * two_pi
    Real.1 * two_pi = two_pi
    (from_nat[Real](k) + Real.1) * two_pi = from_nat[Real](k) * two_pi + two_pi
    from_nat[Real](k.suc) * two_pi = from_nat[Real](k) * two_pi + two_pi
    sin_nt(k.suc, two_pi) = (from_nat[Real](k) * two_pi + two_pi).sin
    sin_add(from_nat[Real](k) * two_pi, two_pi)
    (from_nat[Real](k) * two_pi + two_pi).sin = (from_nat[Real](k) * two_pi).sin * two_pi.cos + (from_nat[Real](k) * two_pi).cos * two_pi.sin

    sin_nt(k, two_pi) = (from_nat[Real](k) * two_pi).sin
    (from_nat[Real](k) * two_pi).sin = sin_nt(k, two_pi)
    cos_nt(k, two_pi) = (from_nat[Real](k) * two_pi).cos
    (from_nat[Real](k) * two_pi).cos = cos_nt(k, two_pi)
    (from_nat[Real](k) * two_pi).sin * two_pi.cos + (from_nat[Real](k) * two_pi).cos * two_pi.sin = sin_nt(k, two_pi) * two_pi.cos + cos_nt(k, two_pi) * two_pi.sin

    sin_nt(k.suc, two_pi) = sin_nt(k, two_pi) * two_pi.cos + cos_nt(k, two_pi) * two_pi.sin
}

/// The sine harmonic vanishes at the period: (k * 2*pi).sin = 0 for all k.
theorem sin_nt_two_pi_zero(k: Nat) {
    sin_nt(k, two_pi) = Real.0
} by {
    define q(m: Nat) -> Bool {
        sin_nt(m, two_pi) = Real.0
    }
    sin_nt_zero_harmonic(two_pi)
    sin_nt(Nat.0, two_pi) = Real.0
    q(Nat.0)
    forall(m: Nat) {
        if q(m) {
            sin_nt_suc_at_two_pi(m)
            sin_nt(m.suc, two_pi) = sin_nt(m, two_pi) * two_pi.cos + cos_nt(m, two_pi) * two_pi.sin
            sin_two_pi_zero
            two_pi.sin = Real.0
            cos_two_pi_one
            two_pi.cos = Real.1
            sin_nt(m, two_pi) * two_pi.cos + cos_nt(m, two_pi) * two_pi.sin = sin_nt(m, two_pi) * Real.1 + cos_nt(m, two_pi) * Real.0

            sin_nt(m, two_pi) * Real.1 = sin_nt(m, two_pi)
            cos_nt(m, two_pi) * Real.0 = Real.0
            sin_nt(m, two_pi) * Real.1 + cos_nt(m, two_pi) * Real.0 = sin_nt(m, two_pi)
            sin_nt(m, two_pi) = Real.0
            sin_nt(m.suc, two_pi) = Real.0
            q(m.suc)
        }
    }
    alt_induction(q)
    (q(Nat.0) and forall(j: Nat) { q(j) implies q(j.suc) }) implies forall(j: Nat) { q(j) }
    q(Nat.0) and forall(j: Nat) { q(j) implies q(j.suc) }
    forall(j: Nat) { q(j) }
    q(k)
}

/// The successor harmonic of cosine at the period: ((k + 1) * 2*pi).cos in
/// terms of (k * 2*pi).cos and (k * 2*pi).sin.
theorem cos_nt_suc_at_two_pi(k: Nat) {
    cos_nt(k.suc, two_pi) = cos_nt(k, two_pi) * two_pi.cos - sin_nt(k, two_pi) * two_pi.sin
} by {
    cos_nt(k.suc, two_pi) = (from_nat[Real](k.suc) * two_pi).cos
    from_nat_add[Real](k, Nat.1)
    from_nat[Real](k + Nat.1) = from_nat[Real](k) + from_nat[Real](Nat.1)
    k + Nat.1 = k.suc
    from_nat[Real](k.suc) = from_nat[Real](k) + from_nat[Real](Nat.1)
    from_nat[Real](Nat.1) = Real.1
    from_nat[Real](k.suc) = from_nat[Real](k) + Real.1
    mul_distrib_left(from_nat[Real](k), Real.1, two_pi)
    (from_nat[Real](k) + Real.1) * two_pi = from_nat[Real](k) * two_pi + Real.1 * two_pi
    Real.1 * two_pi = two_pi
    (from_nat[Real](k) + Real.1) * two_pi = from_nat[Real](k) * two_pi + two_pi
    from_nat[Real](k.suc) * two_pi = from_nat[Real](k) * two_pi + two_pi
    cos_nt(k.suc, two_pi) = (from_nat[Real](k) * two_pi + two_pi).cos
    cos_add(from_nat[Real](k) * two_pi, two_pi)
    (from_nat[Real](k) * two_pi + two_pi).cos = (from_nat[Real](k) * two_pi).cos * two_pi.cos - (from_nat[Real](k) * two_pi).sin * two_pi.sin

    sin_nt(k, two_pi) = (from_nat[Real](k) * two_pi).sin
    (from_nat[Real](k) * two_pi).sin = sin_nt(k, two_pi)
    cos_nt(k, two_pi) = (from_nat[Real](k) * two_pi).cos
    (from_nat[Real](k) * two_pi).cos = cos_nt(k, two_pi)
    (from_nat[Real](k) * two_pi).cos * two_pi.cos - (from_nat[Real](k) * two_pi).sin * two_pi.sin = cos_nt(k, two_pi) * two_pi.cos - sin_nt(k, two_pi) * two_pi.sin

    cos_nt(k.suc, two_pi) = cos_nt(k, two_pi) * two_pi.cos - sin_nt(k, two_pi) * two_pi.sin
}

/// The cosine harmonic is one at the period: (k * 2*pi).cos = 1 for all k.
theorem cos_nt_two_pi_one(k: Nat) {
    cos_nt(k, two_pi) = Real.1
} by {
    define q(m: Nat) -> Bool {
        cos_nt(m, two_pi) = Real.1
    }
    cos_nt_zero_harmonic(two_pi)
    cos_nt(Nat.0, two_pi) = Real.1
    q(Nat.0)
    forall(m: Nat) {
        if q(m) {
            cos_nt_suc_at_two_pi(m)
            cos_nt(m.suc, two_pi) = cos_nt(m, two_pi) * two_pi.cos - sin_nt(m, two_pi) * two_pi.sin
            sin_two_pi_zero
            two_pi.sin = Real.0
            cos_two_pi_one
            two_pi.cos = Real.1
            cos_nt(m, two_pi) * two_pi.cos - sin_nt(m, two_pi) * two_pi.sin = cos_nt(m, two_pi) * Real.1 - sin_nt(m, two_pi) * Real.0

            cos_nt(m, two_pi) * Real.1 = cos_nt(m, two_pi)
            sin_nt(m, two_pi) * Real.0 = Real.0
            cos_nt(m, two_pi) * Real.1 - sin_nt(m, two_pi) * Real.0 = cos_nt(m, two_pi)
            cos_nt(m, two_pi) = Real.1
            cos_nt(m.suc, two_pi) = Real.1
            q(m.suc)
        }
    }
    alt_induction(q)
    (q(Nat.0) and forall(j: Nat) { q(j) implies q(j.suc) }) implies forall(j: Nat) { q(j) }
    q(Nat.0) and forall(j: Nat) { q(j) implies q(j.suc) }
    forall(j: Nat) { q(j) }
    q(k)
}

// ---------------------------------------------------------------------------
// The harmonics are composed with the linear maps x -> kx: the addition
// formulas give the successor decompositions
//   ((k+1)x).sin = (kx).sincos(x) + kx.cos(x).sin, //   ((k+1)x).cos = kx.cos(x).cos - kx.sin(x).sin, // from which the derivatives of the harmonics are obtained by induction.


// ---------------------------------------------------------------------------

/// ((k+1)x).sin = (kx).sincos(x) + kx.cos(x).sin.
theorem sin_nt_suc_decomp(k: Nat, x: Real) {
    sin_nt(k.suc, x) = sin_nt(k, x) * x.cos + cos_nt(k, x) * x.sin
} by {
    sin_nt(k.suc, x) = (from_nat[Real](k.suc) * x).sin
    from_nat_add[Real](k, Nat.1)
    from_nat[Real](k + Nat.1) = from_nat[Real](k) + from_nat[Real](Nat.1)
    k + Nat.1 = k.suc
    from_nat[Real](k.suc) = from_nat[Real](k) + from_nat[Real](Nat.1)
    from_nat[Real](Nat.1) = Real.1
    from_nat[Real](k.suc) = from_nat[Real](k) + Real.1
    mul_distrib_left(from_nat[Real](k), Real.1, x)
    (from_nat[Real](k) + Real.1) * x = from_nat[Real](k) * x + Real.1 * x
    Real.1 * x = x
    (from_nat[Real](k) + Real.1) * x = from_nat[Real](k) * x + x
    from_nat[Real](k.suc) * x = from_nat[Real](k) * x + x
    sin_nt(k.suc, x) = (from_nat[Real](k) * x + x).sin
    sin_add(from_nat[Real](k) * x, x)
    (from_nat[Real](k) * x + x).sin = (from_nat[Real](k) * x).sin * x.cos + (from_nat[Real](k) * x).cos * x.sin

    sin_nt(k, x) = (from_nat[Real](k) * x).sin
    (from_nat[Real](k) * x).sin = sin_nt(k, x)
    cos_nt(k, x) = (from_nat[Real](k) * x).cos
    (from_nat[Real](k) * x).cos = cos_nt(k, x)
    (from_nat[Real](k) * x).sin * x.cos + (from_nat[Real](k) * x).cos * x.sin = sin_nt(k, x) * x.cos + cos_nt(k, x) * x.sin

    sin_nt(k.suc, x) = sin_nt(k, x) * x.cos + cos_nt(k, x) * x.sin
}

/// ((k+1)x).cos = kx.cos(x).cos - kx.sin(x).sin.
theorem cos_nt_suc_decomp(k: Nat, x: Real) {
    cos_nt(k.suc, x) = cos_nt(k, x) * x.cos - sin_nt(k, x) * x.sin
} by {
    cos_nt(k.suc, x) = (from_nat[Real](k.suc) * x).cos
    from_nat_add[Real](k, Nat.1)
    from_nat[Real](k + Nat.1) = from_nat[Real](k) + from_nat[Real](Nat.1)
    k + Nat.1 = k.suc
    from_nat[Real](k.suc) = from_nat[Real](k) + from_nat[Real](Nat.1)
    from_nat[Real](Nat.1) = Real.1
    from_nat[Real](k.suc) = from_nat[Real](k) + Real.1
    mul_distrib_left(from_nat[Real](k), Real.1, x)
    (from_nat[Real](k) + Real.1) * x = from_nat[Real](k) * x + Real.1 * x
    Real.1 * x = x
    (from_nat[Real](k) + Real.1) * x = from_nat[Real](k) * x + x
    from_nat[Real](k.suc) * x = from_nat[Real](k) * x + x
    cos_nt(k.suc, x) = (from_nat[Real](k) * x + x).cos
    cos_add(from_nat[Real](k) * x, x)
    (from_nat[Real](k) * x + x).cos = (from_nat[Real](k) * x).cos * x.cos - (from_nat[Real](k) * x).sin * x.sin

    sin_nt(k, x) = (from_nat[Real](k) * x).sin
    (from_nat[Real](k) * x).sin = sin_nt(k, x)
    cos_nt(k, x) = (from_nat[Real](k) * x).cos
    (from_nat[Real](k) * x).cos = cos_nt(k, x)
    (from_nat[Real](k) * x).cos * x.cos - (from_nat[Real](k) * x).sin * x.sin = cos_nt(k, x) * x.cos - sin_nt(k, x) * x.sin

    cos_nt(k.suc, x) = cos_nt(k, x) * x.cos - sin_nt(k, x) * x.sin
}


// ---------------------------------------------------------------------------
// The derivative of the harmonics: d/dx kx.sin = k kx.cos and // d/dx kx.cos = -k kx.sin, by simultaneous induction on k from the

// successor decompositions and the product rule.
// ---------------------------------------------------------------------------

/// The pointwise derivative expression of ((m+1)x).sin reduces to
/// (m+1) ((m+1)x).cos: the sum of the product-rule outputs.
/// Ring rearrangement used by the sine derivative expression:
/// -a + b + (c - d) = b - d + (c - a).
theorem ring_deriv_sin_rearrange(a: Real, b: Real, c: Real, d: Real) {
    -a + b + (c - d) = b - d + (c - a)
} by {
    add_assoc(-a, b, c - d)
    (-a + b) + (c - d) = -a + (b + (c - d))
    add_comm(-a, b)
    -a + b = b + -a
    (-a + b) + (c - d) = (b + -a) + (c - d)
    add_assoc(b, -a, c - d)
    (b + -a) + (c - d) = b + (-a + (c - d))
    add_assoc(-a, c, -d)
    -a + (c + -d) = (-a + c) + -d
    c - d = c + -d
    -a + (c - d) = -a + (c + -d)
    add_assoc(c, -d, -a)
    (c + -d) + -a = c + (-d + -a)
    add_comm(-d, -a)
    -d + -a = -a + -d
    (c + -d) + -a = c + (-a + -d)
    b + (-a + (c - d)) = b + ((c + -d) + -a)
    add_assoc(b, c, -a + -d)
    b + (c + (-a + -d)) = (b + c) + (-a + -d)
    neg_distrib(a, d)
    -(a + d) = -a + -d
    b + (c + (-a + -d)) = b + (c + -(a + d))
    (-a + b) + (c - d) = b + (c + -(a + d))
    add_assoc(b, c, -(a + d))
    b + (c + -(a + d)) = (b + c) + -(a + d)
    (-a + b) + (c - d) = (b + c) + -(a + d)
    add_comm(b, c)
    b + c = c + b
    (-a + b) + (c - d) = (c + b) + -(a + d)
    add_assoc(c, b, -(a + d))
    (c + b) + -(a + d) = c + (b + -(a + d))
    (-a + b) + (c - d) = c + (b + -(a + d))
    add_assoc(b, -d, c - a)
    (b + -d) + (c - a) = b + (-d + (c - a))
    c - a = c + -a
    b - d = b + -d
    b - d + (c - a) = (b + -d) + (c - a)
    add_comm(-d, c - a)
    -d + (c - a) = (c - a) + -d
    b + (-d + (c - a)) = b + ((c - a) + -d)
    c - a = c + -a
    (c - a) + -d = (c + -a) + -d
    add_assoc(c, -a, -d)
    (c + -a) + -d = c + (-a + -d)
    neg_distrib(a, d)
    -(a + d) = -a + -d
    b + ((c - a) + -d) = b + (c + -(a + d))
    b + (-d + (c - a)) = b + (c + -(a + d))
    b - d + (c - a) = b + (c + -(a + d))
    (-a + b) + (c - d) = b - d + (c - a)
}

/// The pointwise derivative expression of ((m+1)x).sin reduces to
/// (m+1) ((m+1)x).cos: the sum of the product-rule outputs.
theorem sin_deriv_expr_eq(m: Nat, x: Real) {
    (sin_nt(m, x) * -x.sin + x.cos * (from_nat[Real](m) * cos_nt(m, x))) +
        (cos_nt(m, x) * x.cos + x.sin * -(from_nat[Real](m) * sin_nt(m, x))) = from_nat[Real](m.suc) * cos_nt(m.suc, x)

} by {
    sin_nt(m, x) * -x.sin = -(sin_nt(m, x) * x.sin)
    x.cos * (from_nat[Real](m) * cos_nt(m, x)) = from_nat[Real](m) * (cos_nt(m, x) * x.cos)
    real_mul_comm(x.cos, cos_nt(m, x))
    x.cos * cos_nt(m, x) = cos_nt(m, x) * x.cos
    x.cos * (from_nat[Real](m) * cos_nt(m, x)) = from_nat[Real](m) * (x.cos * cos_nt(m, x))
    mul_assoc(from_nat[Real](m), x.cos, cos_nt(m, x))
    from_nat[Real](m) * (x.cos * cos_nt(m, x)) = (from_nat[Real](m) * x.cos) * cos_nt(m, x)
    real_mul_comm(from_nat[Real](m), x.cos)
    from_nat[Real](m) * x.cos = x.cos * from_nat[Real](m)
    (from_nat[Real](m) * x.cos) * cos_nt(m, x) = (x.cos * from_nat[Real](m)) * cos_nt(m, x)
    mul_assoc(x.cos, from_nat[Real](m), cos_nt(m, x))
    (x.cos * from_nat[Real](m)) * cos_nt(m, x) = x.cos * (from_nat[Real](m) * cos_nt(m, x))
    from_nat[Real](m) * (cos_nt(m, x) * x.cos) = x.cos * (from_nat[Real](m) * cos_nt(m, x))
    x.cos * (from_nat[Real](m) * cos_nt(m, x)) = from_nat[Real](m) * (cos_nt(m, x) * x.cos)
    x.sin * -(from_nat[Real](m) * sin_nt(m, x)) = -(x.sin * (from_nat[Real](m) * sin_nt(m, x)))
    real_mul_comm(x.sin, from_nat[Real](m) * sin_nt(m, x))
    x.sin * (from_nat[Real](m) * sin_nt(m, x)) = (from_nat[Real](m) * sin_nt(m, x)) * x.sin
    real_mul_comm(x.sin, sin_nt(m, x))
    x.sin * sin_nt(m, x) = sin_nt(m, x) * x.sin
    from_nat[Real](m) * (sin_nt(m, x) * x.sin) = from_nat[Real](m) * (x.sin * sin_nt(m, x))
    mul_assoc(from_nat[Real](m), sin_nt(m, x), x.sin)
    from_nat[Real](m) * (sin_nt(m, x) * x.sin) = (from_nat[Real](m) * sin_nt(m, x)) * x.sin
    (from_nat[Real](m) * sin_nt(m, x)) * x.sin = from_nat[Real](m) * (x.sin * sin_nt(m, x))
    x.sin * (from_nat[Real](m) * sin_nt(m, x)) = from_nat[Real](m) * (sin_nt(m, x) * x.sin)
    x.sin * -(from_nat[Real](m) * sin_nt(m, x)) = -(from_nat[Real](m) * (sin_nt(m, x) * x.sin))
    (sin_nt(m, x) * -x.sin + x.cos * (from_nat[Real](m) * cos_nt(m, x))) +
        (cos_nt(m, x) * x.cos + x.sin * -(from_nat[Real](m) * sin_nt(m, x))) = -(sin_nt(m, x) * x.sin) + from_nat[Real](m) * (cos_nt(m, x) * x.cos) +

        (cos_nt(m, x) * x.cos - from_nat[Real](m) * (sin_nt(m, x) * x.sin))
    from_nat_add[Real](m, Nat.1)
    from_nat[Real](m + Nat.1) = from_nat[Real](m) + from_nat[Real](Nat.1)
    m + Nat.1 = m.suc
    from_nat[Real](m.suc) = from_nat[Real](m) + from_nat[Real](Nat.1)
    from_nat[Real](Nat.1) = Real.1
    from_nat[Real](m.suc) = from_nat[Real](m) + Real.1
    cos_nt_suc_decomp(m, x)
    cos_nt(m.suc, x) = cos_nt(m, x) * x.cos - sin_nt(m, x) * x.sin
    mul_distrib_left(from_nat[Real](m), Real.1, cos_nt(m, x) * x.cos - sin_nt(m, x) * x.sin)
    (from_nat[Real](m) + Real.1) * (cos_nt(m, x) * x.cos - sin_nt(m, x) * x.sin) = from_nat[Real](m) * (cos_nt(m, x) * x.cos - sin_nt(m, x) * x.sin) +

        Real.1 * (cos_nt(m, x) * x.cos - sin_nt(m, x) * x.sin)
    Real.1 * (cos_nt(m, x) * x.cos - sin_nt(m, x) * x.sin) = cos_nt(m, x) * x.cos - sin_nt(m, x) * x.sin

    from_nat[Real](m.suc) * cos_nt(m.suc, x) = from_nat[Real](m) * (cos_nt(m, x) * x.cos - sin_nt(m, x) * x.sin) +

        (cos_nt(m, x) * x.cos - sin_nt(m, x) * x.sin)
    mul_sub_distrib_left(from_nat[Real](m), cos_nt(m, x) * x.cos, sin_nt(m, x) * x.sin)
    from_nat[Real](m) * (cos_nt(m, x) * x.cos - sin_nt(m, x) * x.sin) = from_nat[Real](m) * (cos_nt(m, x) * x.cos) - from_nat[Real](m) * (sin_nt(m, x) * x.sin)

    from_nat[Real](m.suc) * cos_nt(m.suc, x) = from_nat[Real](m) * (cos_nt(m, x) * x.cos) - from_nat[Real](m) * (sin_nt(m, x) * x.sin) +

        (cos_nt(m, x) * x.cos - sin_nt(m, x) * x.sin)
    // The two sides of the ring rearrangement agree.
    ring_deriv_sin_rearrange(sin_nt(m, x) * x.sin, from_nat[Real](m) * (cos_nt(m, x) * x.cos), cos_nt(m, x) * x.cos, from_nat[Real](m) * (sin_nt(m, x) * x.sin))

    -(sin_nt(m, x) * x.sin) + from_nat[Real](m) * (cos_nt(m, x) * x.cos) +
        (cos_nt(m, x) * x.cos - from_nat[Real](m) * (sin_nt(m, x) * x.sin)) = from_nat[Real](m) * (cos_nt(m, x) * x.cos) - from_nat[Real](m) * (sin_nt(m, x) * x.sin) +

        (cos_nt(m, x) * x.cos - sin_nt(m, x) * x.sin)
    (sin_nt(m, x) * -x.sin + x.cos * (from_nat[Real](m) * cos_nt(m, x))) +
        (cos_nt(m, x) * x.cos + x.sin * -(from_nat[Real](m) * sin_nt(m, x))) = from_nat[Real](m.suc) * cos_nt(m.suc, x)

}

/// Negation distributes over a sum in the reverse direction:
/// -a + -b = -(a + b).
theorem ring_neg_sum(a: Real, b: Real) {
    -a + -b = -(a + b)
} by {
    neg_distrib(a, b)
    -(a + b) = -a + -b
    -a + -b = -(a + b)
}

/// Ring rearrangement used by the cosine derivative expression:
/// (v + k u) + (u + k v) = k u + k v + u + v.
theorem ring_cos_inside(k: Real, u: Real, v: Real) {
    (v + k * u) + (u + k * v) = k * u + k * v + u + v
} by {
    add_comm(v, k * u)
    v + k * u = k * u + v
    (v + k * u) + (u + k * v) = (k * u + v) + (u + k * v)
    add_assoc(k * u, v, u + k * v)
    (k * u + v) + (u + k * v) = k * u + (v + (u + k * v))
    add_comm(v, u + k * v)
    v + (u + k * v) = (u + k * v) + v
    k * u + (v + (u + k * v)) = k * u + ((u + k * v) + v)
    add_assoc(u, k * v, v)
    (u + k * v) + v = u + (k * v + v)
    k * u + ((u + k * v) + v) = k * u + (u + (k * v + v))
    add_assoc(k * u, u, k * v + v)
    k * u + (u + (k * v + v)) = (k * u + u) + (k * v + v)
    (v + k * u) + (u + k * v) = (k * u + u) + (k * v + v)
    add_comm(k * u, u)
    k * u + u = u + k * u
    add_comm(k * v, v)
    k * v + v = v + k * v
    (k * u + u) + (k * v + v) = (u + k * u) + (v + k * v)
    (v + k * u) + (u + k * v) = (u + k * u) + (v + k * v)
    add_assoc(u, k * u, v + k * v)
    (u + k * u) + (v + k * v) = u + (k * u + (v + k * v))
    add_comm(k * u, v + k * v)
    k * u + (v + k * v) = (v + k * v) + k * u
    u + (k * u + (v + k * v)) = u + ((v + k * v) + k * u)
    add_assoc(u, v + k * v, k * u)
    u + ((v + k * v) + k * u) = (u + (v + k * v)) + k * u
    add_assoc(u, v, k * v)
    (u + v) + k * v = u + (v + k * v)
    (u + (v + k * v)) + k * u = ((u + v) + k * v) + k * u
    add_assoc(u + v, k * v, k * u)
    ((u + v) + k * v) + k * u = (u + v) + (k * v + k * u)
    add_comm(k * v, k * u)
    k * v + k * u = k * u + k * v
    (u + v) + (k * v + k * u) = (u + v) + (k * u + k * v)
    add_comm(u + v, k * u + k * v)
    (u + v) + (k * u + k * v) = (k * u + k * v) + (u + v)
    (v + k * u) + (u + k * v) = (k * u + k * v) + (u + v)
    (k * u + k * v) + (u + v) = k * u + k * v + u + v
    (v + k * u) + (u + k * v) = k * u + k * v + u + v
}


/// Ring rearrangement used by the cosine derivative expression:
/// (-v) + (-(k u)) + (-(u + k v)) = -(k u + k v + u + v).
theorem ring_deriv_cos_rearrange(k: Real, u: Real, v: Real) {
    (-v + -(k * u)) + -(u + k * v) = -(k * u + k * v + u + v)
} by {
    neg_distrib(v, k * u)
    -(v + k * u) = -v + -(k * u)
    ring_neg_sum(v + k * u, u + k * v)
    -(v + k * u) + -(u + k * v) = -((v + k * u) + (u + k * v))
    ring_cos_inside(k, u, v)
    (v + k * u) + (u + k * v) = k * u + k * v + u + v
    -((v + k * u) + (u + k * v)) = -(k * u + k * v + u + v)
    (-v + -(k * u)) + -(u + k * v) = -(k * u + k * v + u + v)
}

/// The pointwise derivative expression of ((m+1)x).cos reduces to
/// -(m+1) ((m+1)x).sin: the sum of the product-rule outputs.
theorem cos_deriv_expr_eq(m: Nat, x: Real) {
    (cos_nt(m, x) * -x.sin + x.cos * -(from_nat[Real](m) * sin_nt(m, x))) +
        -(sin_nt(m, x) * x.cos + x.sin * (from_nat[Real](m) * cos_nt(m, x))) = -(from_nat[Real](m.suc) * sin_nt(m.suc, x))

} by {
    cos_nt(m, x) * -x.sin = -(cos_nt(m, x) * x.sin)
    x.cos * -(from_nat[Real](m) * sin_nt(m, x)) = -(x.cos * (from_nat[Real](m) * sin_nt(m, x)))
    real_mul_comm(x.cos, from_nat[Real](m) * sin_nt(m, x))
    x.cos * (from_nat[Real](m) * sin_nt(m, x)) = (from_nat[Real](m) * sin_nt(m, x)) * x.cos
    real_mul_comm(sin_nt(m, x), x.cos)
    sin_nt(m, x) * x.cos = x.cos * sin_nt(m, x)
    from_nat[Real](m) * (sin_nt(m, x) * x.cos) = from_nat[Real](m) * (x.cos * sin_nt(m, x))
    mul_assoc(from_nat[Real](m), sin_nt(m, x), x.cos)
    from_nat[Real](m) * (sin_nt(m, x) * x.cos) = (from_nat[Real](m) * sin_nt(m, x)) * x.cos
    (from_nat[Real](m) * sin_nt(m, x)) * x.cos = from_nat[Real](m) * (x.cos * sin_nt(m, x))
    x.cos * (from_nat[Real](m) * sin_nt(m, x)) = from_nat[Real](m) * (sin_nt(m, x) * x.cos)
    x.cos * -(from_nat[Real](m) * sin_nt(m, x)) = -(from_nat[Real](m) * (sin_nt(m, x) * x.cos))
    x.sin * (from_nat[Real](m) * cos_nt(m, x)) = from_nat[Real](m) * (x.sin * cos_nt(m, x))
    real_mul_comm(x.sin, cos_nt(m, x))
    x.sin * cos_nt(m, x) = cos_nt(m, x) * x.sin
    mul_assoc(from_nat[Real](m), cos_nt(m, x), x.sin)
    from_nat[Real](m) * (cos_nt(m, x) * x.sin) = (from_nat[Real](m) * cos_nt(m, x)) * x.sin
    real_mul_comm(from_nat[Real](m) * cos_nt(m, x), x.sin)
    (from_nat[Real](m) * cos_nt(m, x)) * x.sin = x.sin * (from_nat[Real](m) * cos_nt(m, x))
    x.sin * (from_nat[Real](m) * cos_nt(m, x)) = from_nat[Real](m) * (cos_nt(m, x) * x.sin)
    (cos_nt(m, x) * -x.sin + x.cos * -(from_nat[Real](m) * sin_nt(m, x))) +
        -(sin_nt(m, x) * x.cos + x.sin * (from_nat[Real](m) * cos_nt(m, x))) = -(cos_nt(m, x) * x.sin) + -(from_nat[Real](m) * (sin_nt(m, x) * x.cos)) +

        -(sin_nt(m, x) * x.cos + from_nat[Real](m) * (cos_nt(m, x) * x.sin))
    from_nat_add[Real](m, Nat.1)
    from_nat[Real](m + Nat.1) = from_nat[Real](m) + from_nat[Real](Nat.1)
    m + Nat.1 = m.suc
    from_nat[Real](m.suc) = from_nat[Real](m) + from_nat[Real](Nat.1)
    from_nat[Real](Nat.1) = Real.1
    from_nat[Real](m.suc) = from_nat[Real](m) + Real.1
    sin_nt_suc_decomp(m, x)
    sin_nt(m.suc, x) = sin_nt(m, x) * x.cos + cos_nt(m, x) * x.sin
    mul_distrib_left(from_nat[Real](m), Real.1, sin_nt(m, x) * x.cos + cos_nt(m, x) * x.sin)
    (from_nat[Real](m) + Real.1) * (sin_nt(m, x) * x.cos + cos_nt(m, x) * x.sin) = from_nat[Real](m) * (sin_nt(m, x) * x.cos + cos_nt(m, x) * x.sin) +

        Real.1 * (sin_nt(m, x) * x.cos + cos_nt(m, x) * x.sin)
    Real.1 * (sin_nt(m, x) * x.cos + cos_nt(m, x) * x.sin) = sin_nt(m, x) * x.cos + cos_nt(m, x) * x.sin

    from_nat[Real](m.suc) * sin_nt(m.suc, x) = from_nat[Real](m) * (sin_nt(m, x) * x.cos + cos_nt(m, x) * x.sin) +

        (sin_nt(m, x) * x.cos + cos_nt(m, x) * x.sin)
    mul_distrib_left(from_nat[Real](m), sin_nt(m, x) * x.cos, cos_nt(m, x) * x.sin)
    from_nat[Real](m) * (sin_nt(m, x) * x.cos + cos_nt(m, x) * x.sin) = from_nat[Real](m) * (sin_nt(m, x) * x.cos) + from_nat[Real](m) * (cos_nt(m, x) * x.sin)

    from_nat[Real](m.suc) * sin_nt(m.suc, x) = from_nat[Real](m) * (sin_nt(m, x) * x.cos) + from_nat[Real](m) * (cos_nt(m, x) * x.sin) +

        (sin_nt(m, x) * x.cos + cos_nt(m, x) * x.sin)
    // The two sides are both the negation of the expanded sum.
    ring_deriv_cos_rearrange(from_nat[Real](m), sin_nt(m, x) * x.cos, cos_nt(m, x) * x.sin)
    -(cos_nt(m, x) * x.sin) + -(from_nat[Real](m) * (sin_nt(m, x) * x.cos)) +
        -(sin_nt(m, x) * x.cos + from_nat[Real](m) * (cos_nt(m, x) * x.sin)) = -(from_nat[Real](m) * (sin_nt(m, x) * x.cos) + from_nat[Real](m) * (cos_nt(m, x) * x.sin) +

            sin_nt(m, x) * x.cos + cos_nt(m, x) * x.sin)
    -(from_nat[Real](m) * (sin_nt(m, x) * x.cos) + from_nat[Real](m) * (cos_nt(m, x) * x.sin) +
        sin_nt(m, x) * x.cos + cos_nt(m, x) * x.sin) = -(from_nat[Real](m.suc) * sin_nt(m.suc, x))

    (cos_nt(m, x) * -x.sin + x.cos * -(from_nat[Real](m) * sin_nt(m, x))) +
        -(sin_nt(m, x) * x.cos + x.sin * (from_nat[Real](m) * cos_nt(m, x))) = -(from_nat[Real](m.suc) * sin_nt(m.suc, x))

}

// ---------------------------------------------------------------------------
// Continuity of the harmonics, by induction on k from the successor // decompositions (composition continuity is not exported by the real

// interface, so the addition formulas are used instead).
// ---------------------------------------------------------------------------

/// The k-th sine harmonic kx.sin is continuous.
theorem sin_nt_continuous(k: Nat) {
    continuous(sin_nt(k))
} by {
    define q(m: Nat) -> Bool {
        continuous(sin_nt(m)) and continuous(cos_nt(m))
    }
    // base case: (0x).sin = 0 and (0x).cos = 1 are constants.
    forall(x: Real) {
        sin_nt_zero_harmonic(x)
        sin_nt(Nat.0, x) = Real.0
        constant[Real, Real](Real.0, x) = Real.0
        sin_nt(Nat.0, x) = constant[Real, Real](Real.0, x)
    }
    function_extensionality(sin_nt(Nat.0), constant[Real, Real](Real.0))
    sin_nt(Nat.0) = constant[Real, Real](Real.0)
    constant_function_is_continuous(Real.0)
    continuous(constant[Real, Real](Real.0))
    continuous(sin_nt(Nat.0))
    forall(x: Real) {
        cos_nt_zero_harmonic(x)
        cos_nt(Nat.0, x) = Real.1
        constant[Real, Real](Real.1, x) = Real.1
        cos_nt(Nat.0, x) = constant[Real, Real](Real.1, x)
    }
    function_extensionality(cos_nt(Nat.0), constant[Real, Real](Real.1))
    cos_nt(Nat.0) = constant[Real, Real](Real.1)
    constant_function_is_continuous(Real.1)
    continuous(constant[Real, Real](Real.1))
    continuous(cos_nt(Nat.0))
    q(Nat.0)
    // inductive step
    forall(m: Nat) {
        if q(m) {
            continuous(sin_nt(m))
            continuous(cos_nt(m))
            sin_continuous
            continuous(Real.sin)
            cos_continuous
            continuous(Real.cos)
            continuous_pointwise_mul(sin_nt(m), Real.cos)
            continuous(pointwise_mul[Real, Real](sin_nt(m), Real.cos))
            continuous_pointwise_mul(cos_nt(m), Real.sin)
            continuous(pointwise_mul[Real, Real](cos_nt(m), Real.sin))
            continuous_pointwise_add(pointwise_mul[Real, Real](sin_nt(m), Real.cos), pointwise_mul[Real, Real](cos_nt(m), Real.sin))

            continuous(pointwise_add(pointwise_mul[Real, Real](sin_nt(m), Real.cos), pointwise_mul[Real, Real](cos_nt(m), Real.sin)))

            forall(x: Real) {
                pointwise_mul_apply(sin_nt(m), Real.cos, x)
                pointwise_mul[Real, Real](sin_nt(m), Real.cos, x) = sin_nt(m, x) * x.cos
                pointwise_mul_apply(cos_nt(m), Real.sin, x)
                pointwise_mul[Real, Real](cos_nt(m), Real.sin, x) = cos_nt(m, x) * x.sin
                pointwise_add_apply(pointwise_mul[Real, Real](sin_nt(m), Real.cos), pointwise_mul[Real, Real](cos_nt(m), Real.sin), x)

                pointwise_add(pointwise_mul[Real, Real](sin_nt(m), Real.cos), pointwise_mul[Real, Real](cos_nt(m), Real.sin), x) = pointwise_mul[Real, Real](sin_nt(m), Real.cos, x) +


                        pointwise_mul[Real, Real](cos_nt(m), Real.sin, x)
                pointwise_add(pointwise_mul[Real, Real](sin_nt(m), Real.cos), pointwise_mul[Real, Real](cos_nt(m), Real.sin), x) = sin_nt(m, x) * x.cos + cos_nt(m, x) * x.sin


                sin_nt_suc_decomp(m, x)
                sin_nt(m.suc, x) = sin_nt(m, x) * x.cos + cos_nt(m, x) * x.sin
                pointwise_add(pointwise_mul[Real, Real](sin_nt(m), Real.cos), pointwise_mul[Real, Real](cos_nt(m), Real.sin), x) = sin_nt(m.suc, x)

            }
            function_extensionality(pointwise_add(pointwise_mul[Real, Real](sin_nt(m), Real.cos), pointwise_mul[Real, Real](cos_nt(m), Real.sin)), sin_nt(m.suc))

            pointwise_add(pointwise_mul[Real, Real](sin_nt(m), Real.cos), pointwise_mul[Real, Real](cos_nt(m), Real.sin)) = sin_nt(m.suc)

            continuous(sin_nt(m.suc))
            // cos_nt(m+1) = cos_nt(m) Real.cos - sin_nt(m) Real.sin is a combination of
            // continuous functions as well.
            continuous_pointwise_neg(pointwise_mul[Real, Real](sin_nt(m), Real.sin))
            continuous(pointwise_neg(pointwise_mul[Real, Real](sin_nt(m), Real.sin)))
            continuous_pointwise_add(pointwise_mul[Real, Real](cos_nt(m), Real.cos), pointwise_neg(pointwise_mul[Real, Real](sin_nt(m), Real.sin)))

            continuous(pointwise_add(pointwise_mul[Real, Real](cos_nt(m), Real.cos), pointwise_neg(pointwise_mul[Real, Real](sin_nt(m), Real.sin))))

            forall(x: Real) {
                pointwise_mul_apply(cos_nt(m), Real.cos, x)
                pointwise_mul[Real, Real](cos_nt(m), Real.cos, x) = cos_nt(m, x) * x.cos
                pointwise_mul_apply(sin_nt(m), Real.sin, x)
                pointwise_mul[Real, Real](sin_nt(m), Real.sin, x) = sin_nt(m, x) * x.sin
                pointwise_neg_apply(pointwise_mul[Real, Real](sin_nt(m), Real.sin), x)
                pointwise_neg(pointwise_mul[Real, Real](sin_nt(m), Real.sin), x) = -(pointwise_mul[Real, Real](sin_nt(m), Real.sin, x))

                pointwise_neg(pointwise_mul[Real, Real](sin_nt(m), Real.sin), x) = -(sin_nt(m, x) * x.sin)

                pointwise_add_apply(pointwise_mul[Real, Real](cos_nt(m), Real.cos), pointwise_neg(pointwise_mul[Real, Real](sin_nt(m), Real.sin)), x)

                pointwise_add(pointwise_mul[Real, Real](cos_nt(m), Real.cos), pointwise_neg(pointwise_mul[Real, Real](sin_nt(m), Real.sin)), x) = pointwise_mul[Real, Real](cos_nt(m), Real.cos, x) +


                        pointwise_neg(pointwise_mul[Real, Real](sin_nt(m), Real.sin), x)
                pointwise_add(pointwise_mul[Real, Real](cos_nt(m), Real.cos), pointwise_neg(pointwise_mul[Real, Real](sin_nt(m), Real.sin)), x) = cos_nt(m, x) * x.cos - sin_nt(m, x) * x.sin


                cos_nt_suc_decomp(m, x)
                cos_nt(m.suc, x) = cos_nt(m, x) * x.cos - sin_nt(m, x) * x.sin
                pointwise_add(pointwise_mul[Real, Real](cos_nt(m), Real.cos), pointwise_neg(pointwise_mul[Real, Real](sin_nt(m), Real.sin)), x) = cos_nt(m.suc, x)

            }
            function_extensionality(pointwise_add(pointwise_mul[Real, Real](cos_nt(m), Real.cos), pointwise_neg(pointwise_mul[Real, Real](sin_nt(m), Real.sin))), cos_nt(m.suc))

            pointwise_add(pointwise_mul[Real, Real](cos_nt(m), Real.cos), pointwise_neg(pointwise_mul[Real, Real](sin_nt(m), Real.sin))) = cos_nt(m.suc)

            continuous(cos_nt(m.suc))
            q(m.suc)
        }
    }
    alt_induction(q)
    (q(Nat.0) and forall(j: Nat) { q(j) implies q(j.suc) }) implies forall(j: Nat) { q(j) }
    q(Nat.0) and forall(j: Nat) { q(j) implies q(j.suc) }
    forall(j: Nat) { q(j) }
    q(k)
    continuous(sin_nt(k))
}

/// The k-th cosine harmonic kx.cos is continuous.
theorem cos_nt_continuous(k: Nat) {
    continuous(cos_nt(k))
} by {
    define q(m: Nat) -> Bool {
        continuous(sin_nt(m)) and continuous(cos_nt(m))
    }
    forall(x: Real) {
        sin_nt_zero_harmonic(x)
        sin_nt(Nat.0, x) = Real.0
        constant[Real, Real](Real.0, x) = Real.0
        sin_nt(Nat.0, x) = constant[Real, Real](Real.0, x)
    }
    function_extensionality(sin_nt(Nat.0), constant[Real, Real](Real.0))
    sin_nt(Nat.0) = constant[Real, Real](Real.0)
    constant_function_is_continuous(Real.0)
    continuous(constant[Real, Real](Real.0))
    continuous(sin_nt(Nat.0))
    forall(x: Real) {
        cos_nt_zero_harmonic(x)
        cos_nt(Nat.0, x) = Real.1
        constant[Real, Real](Real.1, x) = Real.1
        cos_nt(Nat.0, x) = constant[Real, Real](Real.1, x)
    }
    function_extensionality(cos_nt(Nat.0), constant[Real, Real](Real.1))
    cos_nt(Nat.0) = constant[Real, Real](Real.1)
    constant_function_is_continuous(Real.1)
    continuous(constant[Real, Real](Real.1))
    continuous(cos_nt(Nat.0))
    q(Nat.0)
    forall(m: Nat) {
        if q(m) {
            continuous(sin_nt(m))
            continuous(cos_nt(m))
            sin_continuous
            continuous(Real.sin)
            cos_continuous
            continuous(Real.cos)
            continuous_pointwise_mul(sin_nt(m), Real.cos)
            continuous(pointwise_mul[Real, Real](sin_nt(m), Real.cos))
            continuous_pointwise_mul(cos_nt(m), Real.sin)
            continuous(pointwise_mul[Real, Real](cos_nt(m), Real.sin))
            continuous_pointwise_add(pointwise_mul[Real, Real](sin_nt(m), Real.cos), pointwise_mul[Real, Real](cos_nt(m), Real.sin))

            continuous(pointwise_add(pointwise_mul[Real, Real](sin_nt(m), Real.cos), pointwise_mul[Real, Real](cos_nt(m), Real.sin)))

            forall(x: Real) {
                pointwise_mul_apply(sin_nt(m), Real.cos, x)
                pointwise_mul[Real, Real](sin_nt(m), Real.cos, x) = sin_nt(m, x) * x.cos
                pointwise_mul_apply(cos_nt(m), Real.sin, x)
                pointwise_mul[Real, Real](cos_nt(m), Real.sin, x) = cos_nt(m, x) * x.sin
                pointwise_add_apply(pointwise_mul[Real, Real](sin_nt(m), Real.cos), pointwise_mul[Real, Real](cos_nt(m), Real.sin), x)

                pointwise_add(pointwise_mul[Real, Real](sin_nt(m), Real.cos), pointwise_mul[Real, Real](cos_nt(m), Real.sin), x) = pointwise_mul[Real, Real](sin_nt(m), Real.cos, x) +


                        pointwise_mul[Real, Real](cos_nt(m), Real.sin, x)
                pointwise_add(pointwise_mul[Real, Real](sin_nt(m), Real.cos), pointwise_mul[Real, Real](cos_nt(m), Real.sin), x) = sin_nt(m, x) * x.cos + cos_nt(m, x) * x.sin


                sin_nt_suc_decomp(m, x)
                sin_nt(m.suc, x) = sin_nt(m, x) * x.cos + cos_nt(m, x) * x.sin
                pointwise_add(pointwise_mul[Real, Real](sin_nt(m), Real.cos), pointwise_mul[Real, Real](cos_nt(m), Real.sin), x) = sin_nt(m.suc, x)

            }
            function_extensionality(pointwise_add(pointwise_mul[Real, Real](sin_nt(m), Real.cos), pointwise_mul[Real, Real](cos_nt(m), Real.sin)), sin_nt(m.suc))

            pointwise_add(pointwise_mul[Real, Real](sin_nt(m), Real.cos), pointwise_mul[Real, Real](cos_nt(m), Real.sin)) = sin_nt(m.suc)

            continuous(sin_nt(m.suc))
            continuous_pointwise_neg(pointwise_mul[Real, Real](sin_nt(m), Real.sin))
            continuous(pointwise_neg(pointwise_mul[Real, Real](sin_nt(m), Real.sin)))
            continuous_pointwise_add(pointwise_mul[Real, Real](cos_nt(m), Real.cos), pointwise_neg(pointwise_mul[Real, Real](sin_nt(m), Real.sin)))

            continuous(pointwise_add(pointwise_mul[Real, Real](cos_nt(m), Real.cos), pointwise_neg(pointwise_mul[Real, Real](sin_nt(m), Real.sin))))

            forall(x: Real) {
                pointwise_mul_apply(cos_nt(m), Real.cos, x)
                pointwise_mul[Real, Real](cos_nt(m), Real.cos, x) = cos_nt(m, x) * x.cos
                pointwise_mul_apply(sin_nt(m), Real.sin, x)
                pointwise_mul[Real, Real](sin_nt(m), Real.sin, x) = sin_nt(m, x) * x.sin
                pointwise_neg_apply(pointwise_mul[Real, Real](sin_nt(m), Real.sin), x)
                pointwise_neg(pointwise_mul[Real, Real](sin_nt(m), Real.sin), x) = -(pointwise_mul[Real, Real](sin_nt(m), Real.sin, x))

                pointwise_neg(pointwise_mul[Real, Real](sin_nt(m), Real.sin), x) = -(sin_nt(m, x) * x.sin)

                pointwise_add_apply(pointwise_mul[Real, Real](cos_nt(m), Real.cos), pointwise_neg(pointwise_mul[Real, Real](sin_nt(m), Real.sin)), x)

                pointwise_add(pointwise_mul[Real, Real](cos_nt(m), Real.cos), pointwise_neg(pointwise_mul[Real, Real](sin_nt(m), Real.sin)), x) = pointwise_mul[Real, Real](cos_nt(m), Real.cos, x) +


                        pointwise_neg(pointwise_mul[Real, Real](sin_nt(m), Real.sin), x)
                pointwise_add(pointwise_mul[Real, Real](cos_nt(m), Real.cos), pointwise_neg(pointwise_mul[Real, Real](sin_nt(m), Real.sin)), x) = cos_nt(m, x) * x.cos - sin_nt(m, x) * x.sin


                cos_nt_suc_decomp(m, x)
                cos_nt(m.suc, x) = cos_nt(m, x) * x.cos - sin_nt(m, x) * x.sin
                pointwise_add(pointwise_mul[Real, Real](cos_nt(m), Real.cos), pointwise_neg(pointwise_mul[Real, Real](sin_nt(m), Real.sin)), x) = cos_nt(m.suc, x)

            }
            function_extensionality(pointwise_add(pointwise_mul[Real, Real](cos_nt(m), Real.cos), pointwise_neg(pointwise_mul[Real, Real](sin_nt(m), Real.sin))), cos_nt(m.suc))

            pointwise_add(pointwise_mul[Real, Real](cos_nt(m), Real.cos), pointwise_neg(pointwise_mul[Real, Real](sin_nt(m), Real.sin))) = cos_nt(m.suc)

            continuous(cos_nt(m.suc))
            q(m.suc)
        }
    }
    alt_induction(q)
    (q(Nat.0) and forall(j: Nat) { q(j) implies q(j.suc) }) implies forall(j: Nat) { q(j) }
    q(Nat.0) and forall(j: Nat) { q(j) implies q(j.suc) }
    forall(j: Nat) { q(j) }
    q(k)
    continuous(cos_nt(k))
}

// ---------------------------------------------------------------------------
// The derivative of the harmonics: simultaneous induction.  The derivative
// expressions produced by the product rule are named as functions so that the
// function-level equalities with k kx.cos (resp. -k kx.sin) can be proved
// once and reused at every induction step.
// ---------------------------------------------------------------------------

/// The product-rule derivative expression of ((m+1)x).sin:
/// mx.sin(-Real.sin x) + x.cos(m mx.cos) + mx.cos(x).cos + x.sin(-m mx.sin).
define sin_deriv_expr_fn(m: Nat) -> (Real -> Real) {
    pointwise_add(pointwise_add(pointwise_mul(sin_nt(m), pointwise_neg(Real.sin)), pointwise_mul(Real.cos, pointwise_mul(constant[Real, Real](from_nat[Real](m)), cos_nt(m)))), pointwise_add(pointwise_mul(cos_nt(m), Real.cos), pointwise_mul(Real.sin, pointwise_neg(pointwise_mul(constant[Real, Real](from_nat[Real](m)), sin_nt(m))))))



}

/// The pointwise value of the sine derivative expression.
theorem sin_deriv_expr_fn_apply(m: Nat, x: Real) {
    sin_deriv_expr_fn(m, x) = (sin_nt(m, x) * -x.sin + x.cos * (from_nat[Real](m) * cos_nt(m, x))) +

        (cos_nt(m, x) * x.cos + x.sin * -(from_nat[Real](m) * sin_nt(m, x)))
} by {
    pointwise_add_apply(pointwise_mul(sin_nt(m), pointwise_neg(Real.sin)), pointwise_mul(Real.cos, pointwise_mul(constant[Real, Real](from_nat[Real](m)), cos_nt(m))), x)

    pointwise_add(pointwise_mul(sin_nt(m), pointwise_neg(Real.sin)), pointwise_mul(Real.cos, pointwise_mul(constant[Real, Real](from_nat[Real](m)), cos_nt(m))), x) = pointwise_mul(sin_nt(m), pointwise_neg(Real.sin), x) +


            pointwise_mul(Real.cos, pointwise_mul(constant[Real, Real](from_nat[Real](m)), cos_nt(m)), x)
    pointwise_mul_apply(sin_nt(m), pointwise_neg(Real.sin), x)
    pointwise_mul(sin_nt(m), pointwise_neg(Real.sin), x) = sin_nt(m, x) * pointwise_neg(Real.sin, x)
    pointwise_neg_apply(Real.sin, x)
    pointwise_neg(Real.sin, x) = -x.sin
    pointwise_mul(sin_nt(m), pointwise_neg(Real.sin), x) = sin_nt(m, x) * -x.sin
    pointwise_mul_apply(Real.cos, pointwise_mul(constant[Real, Real](from_nat[Real](m)), cos_nt(m)), x)
    pointwise_mul(Real.cos, pointwise_mul(constant[Real, Real](from_nat[Real](m)), cos_nt(m)), x) = x.cos * pointwise_mul(constant[Real, Real](from_nat[Real](m)), cos_nt(m), x)

    pointwise_mul_apply(constant[Real, Real](from_nat[Real](m)), cos_nt(m), x)
    pointwise_mul(constant[Real, Real](from_nat[Real](m)), cos_nt(m), x) = constant[Real, Real](from_nat[Real](m), x) * cos_nt(m, x)

    constant[Real, Real](from_nat[Real](m), x) = from_nat[Real](m)
    pointwise_mul(constant[Real, Real](from_nat[Real](m)), cos_nt(m), x) = from_nat[Real](m) * cos_nt(m, x)

    pointwise_mul(Real.cos, pointwise_mul(constant[Real, Real](from_nat[Real](m)), cos_nt(m)), x) = x.cos * (from_nat[Real](m) * cos_nt(m, x))

    pointwise_add(pointwise_mul(sin_nt(m), pointwise_neg(Real.sin)), pointwise_mul(Real.cos, pointwise_mul(constant[Real, Real](from_nat[Real](m)), cos_nt(m))), x) = sin_nt(m, x) * -x.sin + x.cos * (from_nat[Real](m) * cos_nt(m, x))


    pointwise_add_apply(pointwise_mul(cos_nt(m), Real.cos), pointwise_mul(Real.sin, pointwise_neg(pointwise_mul(constant[Real, Real](from_nat[Real](m)), sin_nt(m)))), x)

    pointwise_add(pointwise_mul(cos_nt(m), Real.cos), pointwise_mul(Real.sin, pointwise_neg(pointwise_mul(constant[Real, Real](from_nat[Real](m)), sin_nt(m)))), x) = pointwise_mul(cos_nt(m), Real.cos, x) +


            pointwise_mul(Real.sin, pointwise_neg(pointwise_mul(constant[Real, Real](from_nat[Real](m)), sin_nt(m))), x)
    pointwise_mul_apply(cos_nt(m), Real.cos, x)
    pointwise_mul(cos_nt(m), Real.cos, x) = cos_nt(m, x) * x.cos
    pointwise_mul_apply(Real.sin, pointwise_neg(pointwise_mul(constant[Real, Real](from_nat[Real](m)), sin_nt(m))), x)
    pointwise_mul(Real.sin, pointwise_neg(pointwise_mul(constant[Real, Real](from_nat[Real](m)), sin_nt(m))), x) = x.sin * pointwise_neg(pointwise_mul(constant[Real, Real](from_nat[Real](m)), sin_nt(m)), x)

    pointwise_neg_apply(pointwise_mul(constant[Real, Real](from_nat[Real](m)), sin_nt(m)), x)
    pointwise_neg(pointwise_mul(constant[Real, Real](from_nat[Real](m)), sin_nt(m)), x) = -(pointwise_mul(constant[Real, Real](from_nat[Real](m)), sin_nt(m), x))

    pointwise_mul_apply(constant[Real, Real](from_nat[Real](m)), sin_nt(m), x)
    pointwise_mul(constant[Real, Real](from_nat[Real](m)), sin_nt(m), x) = constant[Real, Real](from_nat[Real](m), x) * sin_nt(m, x)

    constant[Real, Real](from_nat[Real](m), x) = from_nat[Real](m)
    pointwise_mul(constant[Real, Real](from_nat[Real](m)), sin_nt(m), x) = from_nat[Real](m) * sin_nt(m, x)

    pointwise_neg(pointwise_mul(constant[Real, Real](from_nat[Real](m)), sin_nt(m)), x) = -(from_nat[Real](m) * sin_nt(m, x))

    pointwise_mul(Real.sin, pointwise_neg(pointwise_mul(constant[Real, Real](from_nat[Real](m)), sin_nt(m))), x) = x.sin * -(from_nat[Real](m) * sin_nt(m, x))

    pointwise_add(pointwise_mul(cos_nt(m), Real.cos), pointwise_mul(Real.sin, pointwise_neg(pointwise_mul(constant[Real, Real](from_nat[Real](m)), sin_nt(m)))), x) = cos_nt(m, x) * x.cos + x.sin * -(from_nat[Real](m) * sin_nt(m, x))


    pointwise_add_apply(
        pointwise_add(pointwise_mul(sin_nt(m), pointwise_neg(Real.sin)), pointwise_mul(Real.cos, pointwise_mul(constant[Real, Real](from_nat[Real](m)), cos_nt(m)))), pointwise_add(pointwise_mul(cos_nt(m), Real.cos), pointwise_mul(Real.sin, pointwise_neg(pointwise_mul(constant[Real, Real](from_nat[Real](m)), sin_nt(m))))), x)



    sin_deriv_expr_fn(m, x) = (sin_nt(m, x) * -x.sin + x.cos * (from_nat[Real](m) * cos_nt(m, x))) +

        (cos_nt(m, x) * x.cos + x.sin * -(from_nat[Real](m) * sin_nt(m, x)))
}

/// The product-rule derivative expression of ((m+1)x).cos:
/// mx.cos(-Real.sin x) + x.cos(-m mx.sin) - ((mx).sincos(x) + x.sin(m mx.cos)).
define cos_deriv_expr_fn(m: Nat) -> (Real -> Real) {
    pointwise_add(pointwise_add(pointwise_mul(cos_nt(m), pointwise_neg(Real.sin)), pointwise_mul(Real.cos, pointwise_neg(pointwise_mul(constant[Real, Real](from_nat[Real](m)), sin_nt(m))))), pointwise_neg(pointwise_add(pointwise_mul(sin_nt(m), Real.cos), pointwise_mul(Real.sin, pointwise_mul(constant[Real, Real](from_nat[Real](m)), cos_nt(m))))))



}

/// The pointwise value of the cosine derivative expression.
theorem cos_deriv_expr_fn_apply(m: Nat, x: Real) {
    cos_deriv_expr_fn(m, x) = (cos_nt(m, x) * -x.sin + x.cos * -(from_nat[Real](m) * sin_nt(m, x))) +

        -(sin_nt(m, x) * x.cos + x.sin * (from_nat[Real](m) * cos_nt(m, x)))
} by {
    pointwise_add_apply(pointwise_mul(cos_nt(m), pointwise_neg(Real.sin)), pointwise_mul(Real.cos, pointwise_neg(pointwise_mul(constant[Real, Real](from_nat[Real](m)), sin_nt(m)))), x)

    pointwise_add(pointwise_mul(cos_nt(m), pointwise_neg(Real.sin)), pointwise_mul(Real.cos, pointwise_neg(pointwise_mul(constant[Real, Real](from_nat[Real](m)), sin_nt(m)))), x) = pointwise_mul(cos_nt(m), pointwise_neg(Real.sin), x) +


            pointwise_mul(Real.cos, pointwise_neg(pointwise_mul(constant[Real, Real](from_nat[Real](m)), sin_nt(m))), x)
    pointwise_mul_apply(cos_nt(m), pointwise_neg(Real.sin), x)
    pointwise_mul(cos_nt(m), pointwise_neg(Real.sin), x) = cos_nt(m, x) * pointwise_neg(Real.sin, x)
    pointwise_neg_apply(Real.sin, x)
    pointwise_neg(Real.sin, x) = -x.sin
    pointwise_mul(cos_nt(m), pointwise_neg(Real.sin), x) = cos_nt(m, x) * -x.sin
    pointwise_mul_apply(Real.cos, pointwise_neg(pointwise_mul(constant[Real, Real](from_nat[Real](m)), sin_nt(m))), x)
    pointwise_mul(Real.cos, pointwise_neg(pointwise_mul(constant[Real, Real](from_nat[Real](m)), sin_nt(m))), x) = x.cos * pointwise_neg(pointwise_mul(constant[Real, Real](from_nat[Real](m)), sin_nt(m)), x)

    pointwise_neg_apply(pointwise_mul(constant[Real, Real](from_nat[Real](m)), sin_nt(m)), x)
    pointwise_neg(pointwise_mul(constant[Real, Real](from_nat[Real](m)), sin_nt(m)), x) = -(pointwise_mul(constant[Real, Real](from_nat[Real](m)), sin_nt(m), x))

    pointwise_mul_apply(constant[Real, Real](from_nat[Real](m)), sin_nt(m), x)
    pointwise_mul(constant[Real, Real](from_nat[Real](m)), sin_nt(m), x) = constant[Real, Real](from_nat[Real](m), x) * sin_nt(m, x)

    constant[Real, Real](from_nat[Real](m), x) = from_nat[Real](m)
    pointwise_mul(constant[Real, Real](from_nat[Real](m)), sin_nt(m), x) = from_nat[Real](m) * sin_nt(m, x)

    pointwise_neg(pointwise_mul(constant[Real, Real](from_nat[Real](m)), sin_nt(m)), x) = -(from_nat[Real](m) * sin_nt(m, x))

    pointwise_mul(Real.cos, pointwise_neg(pointwise_mul(constant[Real, Real](from_nat[Real](m)), sin_nt(m))), x) = x.cos * -(from_nat[Real](m) * sin_nt(m, x))

    pointwise_add(pointwise_mul(cos_nt(m), pointwise_neg(Real.sin)), pointwise_mul(Real.cos, pointwise_neg(pointwise_mul(constant[Real, Real](from_nat[Real](m)), sin_nt(m)))), x) = cos_nt(m, x) * -x.sin + x.cos * -(from_nat[Real](m) * sin_nt(m, x))


    pointwise_neg_apply(pointwise_add(pointwise_mul(sin_nt(m), Real.cos), pointwise_mul(Real.sin, pointwise_mul(constant[Real, Real](from_nat[Real](m)), cos_nt(m)))), x)

    pointwise_neg(pointwise_add(pointwise_mul(sin_nt(m), Real.cos), pointwise_mul(Real.sin, pointwise_mul(constant[Real, Real](from_nat[Real](m)), cos_nt(m)))), x) = -(pointwise_add(pointwise_mul(sin_nt(m), Real.cos), pointwise_mul(Real.sin, pointwise_mul(constant[Real, Real](from_nat[Real](m)), cos_nt(m))), x))


    pointwise_add_apply(pointwise_mul(sin_nt(m), Real.cos), pointwise_mul(Real.sin, pointwise_mul(constant[Real, Real](from_nat[Real](m)), cos_nt(m))), x)

    pointwise_add(pointwise_mul(sin_nt(m), Real.cos), pointwise_mul(Real.sin, pointwise_mul(constant[Real, Real](from_nat[Real](m)), cos_nt(m))), x) = pointwise_mul(sin_nt(m), Real.cos, x) +


            pointwise_mul(Real.sin, pointwise_mul(constant[Real, Real](from_nat[Real](m)), cos_nt(m)), x)
    pointwise_mul_apply(sin_nt(m), Real.cos, x)
    pointwise_mul(sin_nt(m), Real.cos, x) = sin_nt(m, x) * x.cos
    pointwise_mul_apply(Real.sin, pointwise_mul(constant[Real, Real](from_nat[Real](m)), cos_nt(m)), x)
    pointwise_mul(Real.sin, pointwise_mul(constant[Real, Real](from_nat[Real](m)), cos_nt(m)), x) = x.sin * pointwise_mul(constant[Real, Real](from_nat[Real](m)), cos_nt(m), x)

    pointwise_mul_apply(constant[Real, Real](from_nat[Real](m)), cos_nt(m), x)
    pointwise_mul(constant[Real, Real](from_nat[Real](m)), cos_nt(m), x) = constant[Real, Real](from_nat[Real](m), x) * cos_nt(m, x)

    constant[Real, Real](from_nat[Real](m), x) = from_nat[Real](m)
    pointwise_mul(constant[Real, Real](from_nat[Real](m)), cos_nt(m), x) = from_nat[Real](m) * cos_nt(m, x)

    pointwise_mul(Real.sin, pointwise_mul(constant[Real, Real](from_nat[Real](m)), cos_nt(m)), x) = x.sin * (from_nat[Real](m) * cos_nt(m, x))

    pointwise_add(pointwise_mul(sin_nt(m), Real.cos), pointwise_mul(Real.sin, pointwise_mul(constant[Real, Real](from_nat[Real](m)), cos_nt(m))), x) = sin_nt(m, x) * x.cos + x.sin * (from_nat[Real](m) * cos_nt(m, x))


    pointwise_neg(pointwise_add(pointwise_mul(sin_nt(m), Real.cos), pointwise_mul(Real.sin, pointwise_mul(constant[Real, Real](from_nat[Real](m)), cos_nt(m)))), x) = -(sin_nt(m, x) * x.cos + x.sin * (from_nat[Real](m) * cos_nt(m, x)))


    pointwise_add_apply(
        pointwise_add(pointwise_mul(cos_nt(m), pointwise_neg(Real.sin)), pointwise_mul(Real.cos, pointwise_neg(pointwise_mul(constant[Real, Real](from_nat[Real](m)), sin_nt(m))))), pointwise_neg(pointwise_add(pointwise_mul(sin_nt(m), Real.cos), pointwise_mul(Real.sin, pointwise_mul(constant[Real, Real](from_nat[Real](m)), cos_nt(m))))), x)



    cos_deriv_expr_fn(m, x) = (cos_nt(m, x) * -x.sin + x.cos * -(from_nat[Real](m) * sin_nt(m, x))) +

        -(sin_nt(m, x) * x.cos + x.sin * (from_nat[Real](m) * cos_nt(m, x)))
}

/// The sine product-rule derivative expression equals (m+1) ((m+1)x).cos.
theorem sin_deriv_expr_fn_eq(m: Nat) {
    pointwise_add(pointwise_add(pointwise_mul(sin_nt(m), pointwise_neg(Real.sin)), pointwise_mul(Real.cos, pointwise_mul(constant[Real, Real](from_nat[Real](m)), cos_nt(m)))), pointwise_add(pointwise_mul(cos_nt(m), Real.cos), pointwise_mul(Real.sin, pointwise_neg(pointwise_mul(constant[Real, Real](from_nat[Real](m)), sin_nt(m)))))) = pointwise_mul(constant[Real, Real](from_nat[Real](m.suc)), cos_nt(m.suc))




} by {
    forall(x: Real) {
        sin_deriv_expr_fn_apply(m, x)
        sin_deriv_expr_fn(m, x) = (sin_nt(m, x) * -x.sin + x.cos * (from_nat[Real](m) * cos_nt(m, x))) +

            (cos_nt(m, x) * x.cos + x.sin * -(from_nat[Real](m) * sin_nt(m, x)))
        sin_deriv_expr_eq(m, x)
        (sin_nt(m, x) * -x.sin + x.cos * (from_nat[Real](m) * cos_nt(m, x))) +
            (cos_nt(m, x) * x.cos + x.sin * -(from_nat[Real](m) * sin_nt(m, x))) = from_nat[Real](m.suc) * cos_nt(m.suc, x)

        pointwise_mul_apply(constant[Real, Real](from_nat[Real](m.suc)), cos_nt(m.suc), x)
        pointwise_mul(constant[Real, Real](from_nat[Real](m.suc)), cos_nt(m.suc), x) = constant[Real, Real](from_nat[Real](m.suc), x) * cos_nt(m.suc, x)

        constant[Real, Real](from_nat[Real](m.suc), x) = from_nat[Real](m.suc)
        pointwise_mul(constant[Real, Real](from_nat[Real](m.suc)), cos_nt(m.suc), x) = from_nat[Real](m.suc) * cos_nt(m.suc, x)

        (sin_nt(m, x) * -x.sin + x.cos * (from_nat[Real](m) * cos_nt(m, x))) +
            (cos_nt(m, x) * x.cos + x.sin * -(from_nat[Real](m) * sin_nt(m, x))) = pointwise_mul(constant[Real, Real](from_nat[Real](m.suc)), cos_nt(m.suc), x)

        pointwise_add_apply(pointwise_mul(sin_nt(m), pointwise_neg(Real.sin)), pointwise_mul(Real.cos, pointwise_mul(constant[Real, Real](from_nat[Real](m)), cos_nt(m))), x)

        pointwise_add(pointwise_mul(sin_nt(m), pointwise_neg(Real.sin)), pointwise_mul(Real.cos, pointwise_mul(constant[Real, Real](from_nat[Real](m)), cos_nt(m))), x) = pointwise_mul(sin_nt(m), pointwise_neg(Real.sin), x) +


                pointwise_mul(Real.cos, pointwise_mul(constant[Real, Real](from_nat[Real](m)), cos_nt(m)), x)
        pointwise_mul_apply(sin_nt(m), pointwise_neg(Real.sin), x)
        pointwise_mul(sin_nt(m), pointwise_neg(Real.sin), x) = sin_nt(m, x) * pointwise_neg(Real.sin, x)
        pointwise_neg_apply(Real.sin, x)
        pointwise_neg(Real.sin, x) = -x.sin
        pointwise_mul(sin_nt(m), pointwise_neg(Real.sin), x) = sin_nt(m, x) * -x.sin
        pointwise_mul_apply(Real.cos, pointwise_mul(constant[Real, Real](from_nat[Real](m)), cos_nt(m)), x)
        pointwise_mul(Real.cos, pointwise_mul(constant[Real, Real](from_nat[Real](m)), cos_nt(m)), x) = x.cos * pointwise_mul(constant[Real, Real](from_nat[Real](m)), cos_nt(m), x)

        pointwise_mul_apply(constant[Real, Real](from_nat[Real](m)), cos_nt(m), x)
        pointwise_mul(constant[Real, Real](from_nat[Real](m)), cos_nt(m), x) = constant[Real, Real](from_nat[Real](m), x) * cos_nt(m, x)

        constant[Real, Real](from_nat[Real](m), x) = from_nat[Real](m)
        pointwise_mul(constant[Real, Real](from_nat[Real](m)), cos_nt(m), x) = from_nat[Real](m) * cos_nt(m, x)

        pointwise_mul(Real.cos, pointwise_mul(constant[Real, Real](from_nat[Real](m)), cos_nt(m)), x) = x.cos * (from_nat[Real](m) * cos_nt(m, x))

        pointwise_add(pointwise_mul(sin_nt(m), pointwise_neg(Real.sin)), pointwise_mul(Real.cos, pointwise_mul(constant[Real, Real](from_nat[Real](m)), cos_nt(m))), x) = sin_nt(m, x) * -x.sin + x.cos * (from_nat[Real](m) * cos_nt(m, x))


        pointwise_add_apply(pointwise_mul(cos_nt(m), Real.cos), pointwise_mul(Real.sin, pointwise_neg(pointwise_mul(constant[Real, Real](from_nat[Real](m)), sin_nt(m)))), x)

        pointwise_add(pointwise_mul(cos_nt(m), Real.cos), pointwise_mul(Real.sin, pointwise_neg(pointwise_mul(constant[Real, Real](from_nat[Real](m)), sin_nt(m)))), x) = pointwise_mul(cos_nt(m), Real.cos, x) +


                pointwise_mul(Real.sin, pointwise_neg(pointwise_mul(constant[Real, Real](from_nat[Real](m)), sin_nt(m))), x)
        pointwise_mul_apply(cos_nt(m), Real.cos, x)
        pointwise_mul(cos_nt(m), Real.cos, x) = cos_nt(m, x) * x.cos
        pointwise_mul_apply(Real.sin, pointwise_neg(pointwise_mul(constant[Real, Real](from_nat[Real](m)), sin_nt(m))), x)
        pointwise_mul(Real.sin, pointwise_neg(pointwise_mul(constant[Real, Real](from_nat[Real](m)), sin_nt(m))), x) = x.sin * pointwise_neg(pointwise_mul(constant[Real, Real](from_nat[Real](m)), sin_nt(m)), x)

        pointwise_neg_apply(pointwise_mul(constant[Real, Real](from_nat[Real](m)), sin_nt(m)), x)
        pointwise_neg(pointwise_mul(constant[Real, Real](from_nat[Real](m)), sin_nt(m)), x) = -(pointwise_mul(constant[Real, Real](from_nat[Real](m)), sin_nt(m), x))

        pointwise_mul_apply(constant[Real, Real](from_nat[Real](m)), sin_nt(m), x)
        pointwise_mul(constant[Real, Real](from_nat[Real](m)), sin_nt(m), x) = constant[Real, Real](from_nat[Real](m), x) * sin_nt(m, x)

        constant[Real, Real](from_nat[Real](m), x) = from_nat[Real](m)
        pointwise_mul(constant[Real, Real](from_nat[Real](m)), sin_nt(m), x) = from_nat[Real](m) * sin_nt(m, x)

        pointwise_neg(pointwise_mul(constant[Real, Real](from_nat[Real](m)), sin_nt(m)), x) = -(from_nat[Real](m) * sin_nt(m, x))

        pointwise_mul(Real.sin, pointwise_neg(pointwise_mul(constant[Real, Real](from_nat[Real](m)), sin_nt(m))), x) = x.sin * -(from_nat[Real](m) * sin_nt(m, x))

        pointwise_add(pointwise_mul(cos_nt(m), Real.cos), pointwise_mul(Real.sin, pointwise_neg(pointwise_mul(constant[Real, Real](from_nat[Real](m)), sin_nt(m)))), x) = cos_nt(m, x) * x.cos + x.sin * -(from_nat[Real](m) * sin_nt(m, x))


        pointwise_add_apply(
            pointwise_add(pointwise_mul(sin_nt(m), pointwise_neg(Real.sin)), pointwise_mul(Real.cos, pointwise_mul(constant[Real, Real](from_nat[Real](m)), cos_nt(m)))), pointwise_add(pointwise_mul(cos_nt(m), Real.cos), pointwise_mul(Real.sin, pointwise_neg(pointwise_mul(constant[Real, Real](from_nat[Real](m)), sin_nt(m))))), x)



        pointwise_add(
            pointwise_add(pointwise_mul(sin_nt(m), pointwise_neg(Real.sin)), pointwise_mul(Real.cos, pointwise_mul(constant[Real, Real](from_nat[Real](m)), cos_nt(m)))), pointwise_add(pointwise_mul(cos_nt(m), Real.cos), pointwise_mul(Real.sin, pointwise_neg(pointwise_mul(constant[Real, Real](from_nat[Real](m)), sin_nt(m))))), x) = pointwise_mul(constant[Real, Real](from_nat[Real](m.suc)), cos_nt(m.suc), x)




    }
    function_extensionality(
        pointwise_add(pointwise_add(pointwise_mul(sin_nt(m), pointwise_neg(Real.sin)), pointwise_mul(Real.cos, pointwise_mul(constant[Real, Real](from_nat[Real](m)), cos_nt(m)))), pointwise_add(pointwise_mul(cos_nt(m), Real.cos), pointwise_mul(Real.sin, pointwise_neg(pointwise_mul(constant[Real, Real](from_nat[Real](m)), sin_nt(m)))))), pointwise_mul(constant[Real, Real](from_nat[Real](m.suc)), cos_nt(m.suc)))




    pointwise_add(pointwise_add(pointwise_mul(sin_nt(m), pointwise_neg(Real.sin)), pointwise_mul(Real.cos, pointwise_mul(constant[Real, Real](from_nat[Real](m)), cos_nt(m)))), pointwise_add(pointwise_mul(cos_nt(m), Real.cos), pointwise_mul(Real.sin, pointwise_neg(pointwise_mul(constant[Real, Real](from_nat[Real](m)), sin_nt(m)))))) = pointwise_mul(constant[Real, Real](from_nat[Real](m.suc)), cos_nt(m.suc))




}

/// The cosine product-rule derivative expression equals -(m+1) ((m+1)x).sin.
theorem cos_deriv_expr_fn_eq(m: Nat) {
    pointwise_add(pointwise_add(pointwise_mul(cos_nt(m), pointwise_neg(Real.sin)), pointwise_mul(Real.cos, pointwise_neg(pointwise_mul(constant[Real, Real](from_nat[Real](m)), sin_nt(m))))), pointwise_neg(pointwise_add(pointwise_mul(sin_nt(m), Real.cos), pointwise_mul(Real.sin, pointwise_mul(constant[Real, Real](from_nat[Real](m)), cos_nt(m)))))) = pointwise_neg(pointwise_mul(constant[Real, Real](from_nat[Real](m.suc)), sin_nt(m.suc)))




} by {
    forall(x: Real) {
        cos_deriv_expr_fn_apply(m, x)
        cos_deriv_expr_fn(m, x) = (cos_nt(m, x) * -x.sin + x.cos * -(from_nat[Real](m) * sin_nt(m, x))) +

            -(sin_nt(m, x) * x.cos + x.sin * (from_nat[Real](m) * cos_nt(m, x)))
        cos_deriv_expr_eq(m, x)
        (cos_nt(m, x) * -x.sin + x.cos * -(from_nat[Real](m) * sin_nt(m, x))) +
            -(sin_nt(m, x) * x.cos + x.sin * (from_nat[Real](m) * cos_nt(m, x))) = -(from_nat[Real](m.suc) * sin_nt(m.suc, x))

        pointwise_mul_apply(constant[Real, Real](from_nat[Real](m.suc)), sin_nt(m.suc), x)
        pointwise_mul(constant[Real, Real](from_nat[Real](m.suc)), sin_nt(m.suc), x) = constant[Real, Real](from_nat[Real](m.suc), x) * sin_nt(m.suc, x)

        constant[Real, Real](from_nat[Real](m.suc), x) = from_nat[Real](m.suc)
        pointwise_mul(constant[Real, Real](from_nat[Real](m.suc)), sin_nt(m.suc), x) = from_nat[Real](m.suc) * sin_nt(m.suc, x)

        pointwise_neg_apply(pointwise_mul(constant[Real, Real](from_nat[Real](m.suc)), sin_nt(m.suc)), x)
        pointwise_neg(pointwise_mul(constant[Real, Real](from_nat[Real](m.suc)), sin_nt(m.suc)), x) = -(pointwise_mul(constant[Real, Real](from_nat[Real](m.suc)), sin_nt(m.suc), x))

        pointwise_neg(pointwise_mul(constant[Real, Real](from_nat[Real](m.suc)), sin_nt(m.suc)), x) = -(from_nat[Real](m.suc) * sin_nt(m.suc, x))

        (cos_nt(m, x) * -x.sin + x.cos * -(from_nat[Real](m) * sin_nt(m, x))) +
            -(sin_nt(m, x) * x.cos + x.sin * (from_nat[Real](m) * cos_nt(m, x))) = pointwise_neg(pointwise_mul(constant[Real, Real](from_nat[Real](m.suc)), sin_nt(m.suc)), x)

        pointwise_add_apply(pointwise_mul(cos_nt(m), pointwise_neg(Real.sin)), pointwise_mul(Real.cos, pointwise_neg(pointwise_mul(constant[Real, Real](from_nat[Real](m)), sin_nt(m)))), x)

        pointwise_add(pointwise_mul(cos_nt(m), pointwise_neg(Real.sin)), pointwise_mul(Real.cos, pointwise_neg(pointwise_mul(constant[Real, Real](from_nat[Real](m)), sin_nt(m)))), x) = pointwise_mul(cos_nt(m), pointwise_neg(Real.sin), x) +


                pointwise_mul(Real.cos, pointwise_neg(pointwise_mul(constant[Real, Real](from_nat[Real](m)), sin_nt(m))), x)
        pointwise_mul_apply(cos_nt(m), pointwise_neg(Real.sin), x)
        pointwise_mul(cos_nt(m), pointwise_neg(Real.sin), x) = cos_nt(m, x) * pointwise_neg(Real.sin, x)
        pointwise_neg_apply(Real.sin, x)
        pointwise_neg(Real.sin, x) = -x.sin
        pointwise_mul(cos_nt(m), pointwise_neg(Real.sin), x) = cos_nt(m, x) * -x.sin
        pointwise_mul_apply(Real.cos, pointwise_neg(pointwise_mul(constant[Real, Real](from_nat[Real](m)), sin_nt(m))), x)
        pointwise_mul(Real.cos, pointwise_neg(pointwise_mul(constant[Real, Real](from_nat[Real](m)), sin_nt(m))), x) = x.cos * pointwise_neg(pointwise_mul(constant[Real, Real](from_nat[Real](m)), sin_nt(m)), x)

        pointwise_neg_apply(pointwise_mul(constant[Real, Real](from_nat[Real](m)), sin_nt(m)), x)
        pointwise_neg(pointwise_mul(constant[Real, Real](from_nat[Real](m)), sin_nt(m)), x) = -(pointwise_mul(constant[Real, Real](from_nat[Real](m)), sin_nt(m), x))

        pointwise_mul_apply(constant[Real, Real](from_nat[Real](m)), sin_nt(m), x)
        pointwise_mul(constant[Real, Real](from_nat[Real](m)), sin_nt(m), x) = constant[Real, Real](from_nat[Real](m), x) * sin_nt(m, x)

        constant[Real, Real](from_nat[Real](m), x) = from_nat[Real](m)
        pointwise_mul(constant[Real, Real](from_nat[Real](m)), sin_nt(m), x) = from_nat[Real](m) * sin_nt(m, x)

        pointwise_neg(pointwise_mul(constant[Real, Real](from_nat[Real](m)), sin_nt(m)), x) = -(from_nat[Real](m) * sin_nt(m, x))

        pointwise_mul(Real.cos, pointwise_neg(pointwise_mul(constant[Real, Real](from_nat[Real](m)), sin_nt(m))), x) = x.cos * -(from_nat[Real](m) * sin_nt(m, x))

        pointwise_add(pointwise_mul(cos_nt(m), pointwise_neg(Real.sin)), pointwise_mul(Real.cos, pointwise_neg(pointwise_mul(constant[Real, Real](from_nat[Real](m)), sin_nt(m)))), x) = cos_nt(m, x) * -x.sin + x.cos * -(from_nat[Real](m) * sin_nt(m, x))


        pointwise_neg_apply(pointwise_add(pointwise_mul(sin_nt(m), Real.cos), pointwise_mul(Real.sin, pointwise_mul(constant[Real, Real](from_nat[Real](m)), cos_nt(m)))), x)

        pointwise_neg(pointwise_add(pointwise_mul(sin_nt(m), Real.cos), pointwise_mul(Real.sin, pointwise_mul(constant[Real, Real](from_nat[Real](m)), cos_nt(m)))), x) = -(pointwise_add(pointwise_mul(sin_nt(m), Real.cos), pointwise_mul(Real.sin, pointwise_mul(constant[Real, Real](from_nat[Real](m)), cos_nt(m))), x))


        pointwise_add_apply(pointwise_mul(sin_nt(m), Real.cos), pointwise_mul(Real.sin, pointwise_mul(constant[Real, Real](from_nat[Real](m)), cos_nt(m))), x)

        pointwise_add(pointwise_mul(sin_nt(m), Real.cos), pointwise_mul(Real.sin, pointwise_mul(constant[Real, Real](from_nat[Real](m)), cos_nt(m))), x) = pointwise_mul(sin_nt(m), Real.cos, x) +


                pointwise_mul(Real.sin, pointwise_mul(constant[Real, Real](from_nat[Real](m)), cos_nt(m)), x)
        pointwise_mul_apply(sin_nt(m), Real.cos, x)
        pointwise_mul(sin_nt(m), Real.cos, x) = sin_nt(m, x) * x.cos
        pointwise_mul_apply(Real.sin, pointwise_mul(constant[Real, Real](from_nat[Real](m)), cos_nt(m)), x)
        pointwise_mul(Real.sin, pointwise_mul(constant[Real, Real](from_nat[Real](m)), cos_nt(m)), x) = x.sin * pointwise_mul(constant[Real, Real](from_nat[Real](m)), cos_nt(m), x)

        pointwise_mul_apply(constant[Real, Real](from_nat[Real](m)), cos_nt(m), x)
        pointwise_mul(constant[Real, Real](from_nat[Real](m)), cos_nt(m), x) = constant[Real, Real](from_nat[Real](m), x) * cos_nt(m, x)

        constant[Real, Real](from_nat[Real](m), x) = from_nat[Real](m)
        pointwise_mul(constant[Real, Real](from_nat[Real](m)), cos_nt(m), x) = from_nat[Real](m) * cos_nt(m, x)

        pointwise_mul(Real.sin, pointwise_mul(constant[Real, Real](from_nat[Real](m)), cos_nt(m)), x) = x.sin * (from_nat[Real](m) * cos_nt(m, x))

        pointwise_add(pointwise_mul(sin_nt(m), Real.cos), pointwise_mul(Real.sin, pointwise_mul(constant[Real, Real](from_nat[Real](m)), cos_nt(m))), x) = sin_nt(m, x) * x.cos + x.sin * (from_nat[Real](m) * cos_nt(m, x))


        pointwise_neg(pointwise_add(pointwise_mul(sin_nt(m), Real.cos), pointwise_mul(Real.sin, pointwise_mul(constant[Real, Real](from_nat[Real](m)), cos_nt(m)))), x) = -(sin_nt(m, x) * x.cos + x.sin * (from_nat[Real](m) * cos_nt(m, x)))


        pointwise_add_apply(
            pointwise_add(pointwise_mul(cos_nt(m), pointwise_neg(Real.sin)), pointwise_mul(Real.cos, pointwise_neg(pointwise_mul(constant[Real, Real](from_nat[Real](m)), sin_nt(m))))), pointwise_neg(pointwise_add(pointwise_mul(sin_nt(m), Real.cos), pointwise_mul(Real.sin, pointwise_mul(constant[Real, Real](from_nat[Real](m)), cos_nt(m))))), x)



        pointwise_add(
            pointwise_add(pointwise_mul(cos_nt(m), pointwise_neg(Real.sin)), pointwise_mul(Real.cos, pointwise_neg(pointwise_mul(constant[Real, Real](from_nat[Real](m)), sin_nt(m))))), pointwise_neg(pointwise_add(pointwise_mul(sin_nt(m), Real.cos), pointwise_mul(Real.sin, pointwise_mul(constant[Real, Real](from_nat[Real](m)), cos_nt(m))))), x) = pointwise_neg(pointwise_mul(constant[Real, Real](from_nat[Real](m.suc)), sin_nt(m.suc)), x)




    }
    function_extensionality(
        pointwise_add(pointwise_add(pointwise_mul(cos_nt(m), pointwise_neg(Real.sin)), pointwise_mul(Real.cos, pointwise_neg(pointwise_mul(constant[Real, Real](from_nat[Real](m)), sin_nt(m))))), pointwise_neg(pointwise_add(pointwise_mul(sin_nt(m), Real.cos), pointwise_mul(Real.sin, pointwise_mul(constant[Real, Real](from_nat[Real](m)), cos_nt(m)))))), pointwise_neg(pointwise_mul(constant[Real, Real](from_nat[Real](m.suc)), sin_nt(m.suc))))




    pointwise_add(pointwise_add(pointwise_mul(cos_nt(m), pointwise_neg(Real.sin)), pointwise_mul(Real.cos, pointwise_neg(pointwise_mul(constant[Real, Real](from_nat[Real](m)), sin_nt(m))))), pointwise_neg(pointwise_add(pointwise_mul(sin_nt(m), Real.cos), pointwise_mul(Real.sin, pointwise_mul(constant[Real, Real](from_nat[Real](m)), cos_nt(m)))))) = pointwise_neg(pointwise_mul(constant[Real, Real](from_nat[Real](m.suc)), sin_nt(m.suc)))




}

/// The pointwise derivative of kx.sin is k kx.cos.
theorem sin_nt_derivative(k: Nat) {
    is_derivative_fn(sin_nt(k), pointwise_mul(constant[Real, Real](from_nat[Real](k)), cos_nt(k)))
} by {
    define q(m: Nat) -> Bool {
        is_derivative_fn(sin_nt(m), pointwise_mul(constant[Real, Real](from_nat[Real](m)), cos_nt(m)))
        and is_derivative_fn(cos_nt(m), pointwise_neg(pointwise_mul(constant[Real, Real](from_nat[Real](m)), sin_nt(m))))

    }
    forall(x: Real) {
        sin_nt_zero_harmonic(x)
        sin_nt(Nat.0, x) = Real.0
        constant[Real, Real](Real.0, x) = Real.0
        sin_nt(Nat.0, x) = constant[Real, Real](Real.0, x)
    }
    function_extensionality(sin_nt(Nat.0), constant[Real, Real](Real.0))
    sin_nt(Nat.0) = constant[Real, Real](Real.0)
    derivative_fn_constant(Real.0)
    is_derivative_fn(constant[Real, Real](Real.0), constant[Real, Real](Real.0))
    forall(x: Real) {
        pointwise_mul_apply(constant[Real, Real](from_nat[Real](Nat.0)), cos_nt(Nat.0), x)
        pointwise_mul(constant[Real, Real](from_nat[Real](Nat.0)), cos_nt(Nat.0), x) = constant[Real, Real](from_nat[Real](Nat.0), x) * cos_nt(Nat.0, x)

        from_nat_zero[Real]
        from_nat[Real](Nat.0) = Real.0
        constant[Real, Real](from_nat[Real](Nat.0), x) = Real.0
        pointwise_mul(constant[Real, Real](from_nat[Real](Nat.0)), cos_nt(Nat.0), x) = Real.0 * cos_nt(Nat.0, x)

        Real.0 * cos_nt(Nat.0, x) = Real.0
        pointwise_mul(constant[Real, Real](from_nat[Real](Nat.0)), cos_nt(Nat.0), x) = Real.0
        constant[Real, Real](Real.0, x) = Real.0
        pointwise_mul(constant[Real, Real](from_nat[Real](Nat.0)), cos_nt(Nat.0), x) = constant[Real, Real](Real.0, x)

    }
    function_extensionality(pointwise_mul(constant[Real, Real](from_nat[Real](Nat.0)), cos_nt(Nat.0)), constant[Real, Real](Real.0))

    pointwise_mul(constant[Real, Real](from_nat[Real](Nat.0)), cos_nt(Nat.0)) = constant[Real, Real](Real.0)
    is_derivative_fn(sin_nt(Nat.0), pointwise_mul(constant[Real, Real](from_nat[Real](Nat.0)), cos_nt(Nat.0)))
    forall(x: Real) {
        cos_nt_zero_harmonic(x)
        cos_nt(Nat.0, x) = Real.1
        constant[Real, Real](Real.1, x) = Real.1
        cos_nt(Nat.0, x) = constant[Real, Real](Real.1, x)
    }
    function_extensionality(cos_nt(Nat.0), constant[Real, Real](Real.1))
    cos_nt(Nat.0) = constant[Real, Real](Real.1)
    derivative_fn_constant(Real.1)
    is_derivative_fn(constant[Real, Real](Real.1), constant[Real, Real](Real.0))
    forall(x: Real) {
        pointwise_mul_apply(constant[Real, Real](from_nat[Real](Nat.0)), sin_nt(Nat.0), x)
        pointwise_mul(constant[Real, Real](from_nat[Real](Nat.0)), sin_nt(Nat.0), x) = constant[Real, Real](from_nat[Real](Nat.0), x) * sin_nt(Nat.0, x)

        from_nat_zero[Real]
        from_nat[Real](Nat.0) = Real.0
        constant[Real, Real](from_nat[Real](Nat.0), x) = Real.0
        pointwise_mul(constant[Real, Real](from_nat[Real](Nat.0)), sin_nt(Nat.0), x) = Real.0 * sin_nt(Nat.0, x)

        Real.0 * sin_nt(Nat.0, x) = Real.0
        pointwise_mul(constant[Real, Real](from_nat[Real](Nat.0)), sin_nt(Nat.0), x) = Real.0
        constant[Real, Real](Real.0, x) = Real.0
        pointwise_mul(constant[Real, Real](from_nat[Real](Nat.0)), sin_nt(Nat.0), x) = constant[Real, Real](Real.0, x)

        pointwise_neg_apply(pointwise_mul(constant[Real, Real](from_nat[Real](Nat.0)), sin_nt(Nat.0)), x)
        pointwise_neg(pointwise_mul(constant[Real, Real](from_nat[Real](Nat.0)), sin_nt(Nat.0)), x) = -(pointwise_mul(constant[Real, Real](from_nat[Real](Nat.0)), sin_nt(Nat.0), x))

        pointwise_neg(pointwise_mul(constant[Real, Real](from_nat[Real](Nat.0)), sin_nt(Nat.0)), x) = -(Real.0)

        neg_zero
        -(Real.0) = Real.0
        pointwise_neg(pointwise_mul(constant[Real, Real](from_nat[Real](Nat.0)), sin_nt(Nat.0)), x) = Real.0

        constant[Real, Real](Real.0, x) = Real.0
        pointwise_neg(pointwise_mul(constant[Real, Real](from_nat[Real](Nat.0)), sin_nt(Nat.0)), x) = constant[Real, Real](Real.0, x)

    }
    function_extensionality(pointwise_neg(pointwise_mul(constant[Real, Real](from_nat[Real](Nat.0)), sin_nt(Nat.0))), constant[Real, Real](Real.0))

    pointwise_neg(pointwise_mul(constant[Real, Real](from_nat[Real](Nat.0)), sin_nt(Nat.0))) = constant[Real, Real](Real.0)

    is_derivative_fn(cos_nt(Nat.0), pointwise_neg(pointwise_mul(constant[Real, Real](from_nat[Real](Nat.0)), sin_nt(Nat.0))))

    q(Nat.0)
    forall(m: Nat) {
        if q(m) {
            is_derivative_fn(sin_nt(m), pointwise_mul(constant[Real, Real](from_nat[Real](m)), cos_nt(m)))
            is_derivative_fn(cos_nt(m), pointwise_neg(pointwise_mul(constant[Real, Real](from_nat[Real](m)), sin_nt(m))))

            // --- sine successor ---
            derivative_fn_mul(sin_nt(m), Real.cos, pointwise_mul(constant[Real, Real](from_nat[Real](m)), cos_nt(m)), pointwise_neg(Real.sin))

            is_derivative_fn(pointwise_mul(sin_nt(m), Real.cos), pointwise_add(pointwise_mul(sin_nt(m), pointwise_neg(Real.sin)), pointwise_mul(Real.cos, pointwise_mul(constant[Real, Real](from_nat[Real](m)), cos_nt(m)))))


            derivative_fn_mul(cos_nt(m), Real.sin, pointwise_neg(pointwise_mul(constant[Real, Real](from_nat[Real](m)), sin_nt(m))), Real.cos)

            is_derivative_fn(pointwise_mul(cos_nt(m), Real.sin), pointwise_add(pointwise_mul(cos_nt(m), Real.cos), pointwise_mul(Real.sin, pointwise_neg(pointwise_mul(constant[Real, Real](from_nat[Real](m)), sin_nt(m))))))


            derivative_fn_add(pointwise_mul(sin_nt(m), Real.cos), pointwise_mul(cos_nt(m), Real.sin), pointwise_add(pointwise_mul(sin_nt(m), pointwise_neg(Real.sin)), pointwise_mul(Real.cos, pointwise_mul(constant[Real, Real](from_nat[Real](m)), cos_nt(m)))), pointwise_add(pointwise_mul(cos_nt(m), Real.cos), pointwise_mul(Real.sin, pointwise_neg(pointwise_mul(constant[Real, Real](from_nat[Real](m)), sin_nt(m))))))




            forall(x: Real) {
                pointwise_mul_apply(sin_nt(m), Real.cos, x)
                pointwise_mul(sin_nt(m), Real.cos, x) = sin_nt(m, x) * x.cos
                pointwise_mul_apply(cos_nt(m), Real.sin, x)
                pointwise_mul(cos_nt(m), Real.sin, x) = cos_nt(m, x) * x.sin
                pointwise_add_apply(pointwise_mul(sin_nt(m), Real.cos), pointwise_mul(cos_nt(m), Real.sin), x)
                pointwise_add(pointwise_mul(sin_nt(m), Real.cos), pointwise_mul(cos_nt(m), Real.sin), x) = pointwise_mul(sin_nt(m), Real.cos, x) + pointwise_mul(cos_nt(m), Real.sin, x)

                pointwise_add(pointwise_mul(sin_nt(m), Real.cos), pointwise_mul(cos_nt(m), Real.sin), x) = sin_nt(m, x) * x.cos + cos_nt(m, x) * x.sin

                sin_nt_suc_decomp(m, x)
                sin_nt(m.suc, x) = sin_nt(m, x) * x.cos + cos_nt(m, x) * x.sin
                pointwise_add(pointwise_mul(sin_nt(m), Real.cos), pointwise_mul(cos_nt(m), Real.sin), x) = sin_nt(m.suc, x)

            }
            function_extensionality(pointwise_add(pointwise_mul(sin_nt(m), Real.cos), pointwise_mul(cos_nt(m), Real.sin)), sin_nt(m.suc))

            pointwise_add(pointwise_mul(sin_nt(m), Real.cos), pointwise_mul(cos_nt(m), Real.sin)) = sin_nt(m.suc)
            is_derivative_fn(sin_nt(m.suc), pointwise_add(

                    pointwise_add(pointwise_mul(sin_nt(m), pointwise_neg(Real.sin)), pointwise_mul(Real.cos, pointwise_mul(constant[Real, Real](from_nat[Real](m)), cos_nt(m)))), pointwise_add(pointwise_mul(cos_nt(m), Real.cos), pointwise_mul(Real.sin, pointwise_neg(pointwise_mul(constant[Real, Real](from_nat[Real](m)), sin_nt(m)))))))



            sin_deriv_expr_fn_eq(m)
            pointwise_add(pointwise_add(pointwise_mul(sin_nt(m), pointwise_neg(Real.sin)), pointwise_mul(Real.cos, pointwise_mul(constant[Real, Real](from_nat[Real](m)), cos_nt(m)))), pointwise_add(pointwise_mul(cos_nt(m), Real.cos), pointwise_mul(Real.sin, pointwise_neg(pointwise_mul(constant[Real, Real](from_nat[Real](m)), sin_nt(m)))))) = pointwise_mul(constant[Real, Real](from_nat[Real](m.suc)), cos_nt(m.suc))




            is_derivative_fn(sin_nt(m.suc), pointwise_mul(constant[Real, Real](from_nat[Real](m.suc)), cos_nt(m.suc)))

            // --- cosine successor ---
            derivative_fn_mul(cos_nt(m), Real.cos, pointwise_neg(pointwise_mul(constant[Real, Real](from_nat[Real](m)), sin_nt(m))), pointwise_neg(Real.sin))


            is_derivative_fn(pointwise_mul(cos_nt(m), Real.cos), pointwise_add(pointwise_mul(cos_nt(m), pointwise_neg(Real.sin)), pointwise_mul(Real.cos, pointwise_neg(pointwise_mul(constant[Real, Real](from_nat[Real](m)), sin_nt(m))))))


            derivative_fn_mul(sin_nt(m), Real.sin, pointwise_mul(constant[Real, Real](from_nat[Real](m)), cos_nt(m)), Real.cos)

            is_derivative_fn(pointwise_mul(sin_nt(m), Real.sin), pointwise_add(pointwise_mul(sin_nt(m), Real.cos), pointwise_mul(Real.sin, pointwise_mul(constant[Real, Real](from_nat[Real](m)), cos_nt(m)))))


            derivative_fn_neg(pointwise_mul(sin_nt(m), Real.sin), pointwise_add(pointwise_mul(sin_nt(m), Real.cos), pointwise_mul(Real.sin, pointwise_mul(constant[Real, Real](from_nat[Real](m)), cos_nt(m)))))


            is_derivative_fn(pointwise_neg(pointwise_mul(sin_nt(m), Real.sin)), pointwise_neg(pointwise_add(pointwise_mul(sin_nt(m), Real.cos), pointwise_mul(Real.sin, pointwise_mul(constant[Real, Real](from_nat[Real](m)), cos_nt(m))))))


            derivative_fn_add(pointwise_mul(cos_nt(m), Real.cos), pointwise_neg(pointwise_mul(sin_nt(m), Real.sin)), pointwise_add(pointwise_mul(cos_nt(m), pointwise_neg(Real.sin)), pointwise_mul(Real.cos, pointwise_neg(pointwise_mul(constant[Real, Real](from_nat[Real](m)), sin_nt(m))))), pointwise_neg(pointwise_add(pointwise_mul(sin_nt(m), Real.cos), pointwise_mul(Real.sin, pointwise_mul(constant[Real, Real](from_nat[Real](m)), cos_nt(m))))))




            forall(x: Real) {
                pointwise_mul_apply(cos_nt(m), Real.cos, x)
                pointwise_mul(cos_nt(m), Real.cos, x) = cos_nt(m, x) * x.cos
                pointwise_mul_apply(sin_nt(m), Real.sin, x)
                pointwise_mul(sin_nt(m), Real.sin, x) = sin_nt(m, x) * x.sin
                pointwise_neg_apply(pointwise_mul(sin_nt(m), Real.sin), x)
                pointwise_neg(pointwise_mul(sin_nt(m), Real.sin), x) = -(pointwise_mul(sin_nt(m), Real.sin, x))
                pointwise_neg(pointwise_mul(sin_nt(m), Real.sin), x) = -(sin_nt(m, x) * x.sin)
                pointwise_add_apply(pointwise_mul(cos_nt(m), Real.cos), pointwise_neg(pointwise_mul(sin_nt(m), Real.sin)), x)
                pointwise_add(pointwise_mul(cos_nt(m), Real.cos), pointwise_neg(pointwise_mul(sin_nt(m), Real.sin)), x) = pointwise_mul(cos_nt(m), Real.cos, x) + pointwise_neg(pointwise_mul(sin_nt(m), Real.sin), x)

                pointwise_add(pointwise_mul(cos_nt(m), Real.cos), pointwise_neg(pointwise_mul(sin_nt(m), Real.sin)), x) = cos_nt(m, x) * x.cos - sin_nt(m, x) * x.sin

                cos_nt_suc_decomp(m, x)
                cos_nt(m.suc, x) = cos_nt(m, x) * x.cos - sin_nt(m, x) * x.sin
                pointwise_add(pointwise_mul(cos_nt(m), Real.cos), pointwise_neg(pointwise_mul(sin_nt(m), Real.sin)), x) = cos_nt(m.suc, x)

            }
            function_extensionality(
                pointwise_add(pointwise_mul(cos_nt(m), Real.cos), pointwise_neg(pointwise_mul(sin_nt(m), Real.sin))), cos_nt(m.suc))

            pointwise_add(pointwise_mul(cos_nt(m), Real.cos), pointwise_neg(pointwise_mul(sin_nt(m), Real.sin))) = cos_nt(m.suc)

            is_derivative_fn(cos_nt(m.suc), pointwise_add(

                    pointwise_add(pointwise_mul(cos_nt(m), pointwise_neg(Real.sin)), pointwise_mul(Real.cos, pointwise_neg(pointwise_mul(constant[Real, Real](from_nat[Real](m)), sin_nt(m))))), pointwise_neg(pointwise_add(pointwise_mul(sin_nt(m), Real.cos), pointwise_mul(Real.sin, pointwise_mul(constant[Real, Real](from_nat[Real](m)), cos_nt(m)))))))



            cos_deriv_expr_fn_eq(m)
            pointwise_add(pointwise_add(pointwise_mul(cos_nt(m), pointwise_neg(Real.sin)), pointwise_mul(Real.cos, pointwise_neg(pointwise_mul(constant[Real, Real](from_nat[Real](m)), sin_nt(m))))), pointwise_neg(pointwise_add(pointwise_mul(sin_nt(m), Real.cos), pointwise_mul(Real.sin, pointwise_mul(constant[Real, Real](from_nat[Real](m)), cos_nt(m)))))) = pointwise_neg(pointwise_mul(constant[Real, Real](from_nat[Real](m.suc)), sin_nt(m.suc)))




            is_derivative_fn(cos_nt(m.suc), pointwise_neg(pointwise_mul(constant[Real, Real](from_nat[Real](m.suc)), sin_nt(m.suc))))

            q(m.suc)
        }
    }
    alt_induction(q)
    (q(Nat.0) and forall(j: Nat) { q(j) implies q(j.suc) }) implies forall(j: Nat) { q(j) }
    q(Nat.0) and forall(j: Nat) { q(j) implies q(j.suc) }
    forall(j: Nat) { q(j) }
    q(k)
    is_derivative_fn(sin_nt(k), pointwise_mul(constant[Real, Real](from_nat[Real](k)), cos_nt(k)))
}

/// The pointwise derivative of kx.cos is -k kx.sin.
theorem cos_nt_derivative(k: Nat) {
    is_derivative_fn(cos_nt(k), pointwise_neg(pointwise_mul(constant[Real, Real](from_nat[Real](k)), sin_nt(k))))
} by {
    define q(m: Nat) -> Bool {
        is_derivative_fn(sin_nt(m), pointwise_mul(constant[Real, Real](from_nat[Real](m)), cos_nt(m)))
        and is_derivative_fn(cos_nt(m), pointwise_neg(pointwise_mul(constant[Real, Real](from_nat[Real](m)), sin_nt(m))))

    }
    forall(x: Real) {
        sin_nt_zero_harmonic(x)
        sin_nt(Nat.0, x) = Real.0
        constant[Real, Real](Real.0, x) = Real.0
        sin_nt(Nat.0, x) = constant[Real, Real](Real.0, x)
    }
    function_extensionality(sin_nt(Nat.0), constant[Real, Real](Real.0))
    sin_nt(Nat.0) = constant[Real, Real](Real.0)
    derivative_fn_constant(Real.0)
    is_derivative_fn(constant[Real, Real](Real.0), constant[Real, Real](Real.0))
    forall(x: Real) {
        pointwise_mul_apply(constant[Real, Real](from_nat[Real](Nat.0)), cos_nt(Nat.0), x)
        pointwise_mul(constant[Real, Real](from_nat[Real](Nat.0)), cos_nt(Nat.0), x) = constant[Real, Real](from_nat[Real](Nat.0), x) * cos_nt(Nat.0, x)

        from_nat_zero[Real]
        from_nat[Real](Nat.0) = Real.0
        constant[Real, Real](from_nat[Real](Nat.0), x) = Real.0
        pointwise_mul(constant[Real, Real](from_nat[Real](Nat.0)), cos_nt(Nat.0), x) = Real.0 * cos_nt(Nat.0, x)

        Real.0 * cos_nt(Nat.0, x) = Real.0
        pointwise_mul(constant[Real, Real](from_nat[Real](Nat.0)), cos_nt(Nat.0), x) = Real.0
        constant[Real, Real](Real.0, x) = Real.0
        pointwise_mul(constant[Real, Real](from_nat[Real](Nat.0)), cos_nt(Nat.0), x) = constant[Real, Real](Real.0, x)

    }
    function_extensionality(pointwise_mul(constant[Real, Real](from_nat[Real](Nat.0)), cos_nt(Nat.0)), constant[Real, Real](Real.0))

    pointwise_mul(constant[Real, Real](from_nat[Real](Nat.0)), cos_nt(Nat.0)) = constant[Real, Real](Real.0)
    is_derivative_fn(sin_nt(Nat.0), pointwise_mul(constant[Real, Real](from_nat[Real](Nat.0)), cos_nt(Nat.0)))
    forall(x: Real) {
        cos_nt_zero_harmonic(x)
        cos_nt(Nat.0, x) = Real.1
        constant[Real, Real](Real.1, x) = Real.1
        cos_nt(Nat.0, x) = constant[Real, Real](Real.1, x)
    }
    function_extensionality(cos_nt(Nat.0), constant[Real, Real](Real.1))
    cos_nt(Nat.0) = constant[Real, Real](Real.1)
    derivative_fn_constant(Real.1)
    is_derivative_fn(constant[Real, Real](Real.1), constant[Real, Real](Real.0))
    forall(x: Real) {
        pointwise_mul_apply(constant[Real, Real](from_nat[Real](Nat.0)), sin_nt(Nat.0), x)
        pointwise_mul(constant[Real, Real](from_nat[Real](Nat.0)), sin_nt(Nat.0), x) = constant[Real, Real](from_nat[Real](Nat.0), x) * sin_nt(Nat.0, x)

        from_nat_zero[Real]
        from_nat[Real](Nat.0) = Real.0
        constant[Real, Real](from_nat[Real](Nat.0), x) = Real.0
        pointwise_mul(constant[Real, Real](from_nat[Real](Nat.0)), sin_nt(Nat.0), x) = Real.0 * sin_nt(Nat.0, x)

        Real.0 * sin_nt(Nat.0, x) = Real.0
        pointwise_mul(constant[Real, Real](from_nat[Real](Nat.0)), sin_nt(Nat.0), x) = Real.0
        constant[Real, Real](Real.0, x) = Real.0
        pointwise_mul(constant[Real, Real](from_nat[Real](Nat.0)), sin_nt(Nat.0), x) = constant[Real, Real](Real.0, x)

        pointwise_neg_apply(pointwise_mul(constant[Real, Real](from_nat[Real](Nat.0)), sin_nt(Nat.0)), x)
        pointwise_neg(pointwise_mul(constant[Real, Real](from_nat[Real](Nat.0)), sin_nt(Nat.0)), x) = -(pointwise_mul(constant[Real, Real](from_nat[Real](Nat.0)), sin_nt(Nat.0), x))

        pointwise_neg(pointwise_mul(constant[Real, Real](from_nat[Real](Nat.0)), sin_nt(Nat.0)), x) = -(Real.0)

        neg_zero
        -(Real.0) = Real.0
        pointwise_neg(pointwise_mul(constant[Real, Real](from_nat[Real](Nat.0)), sin_nt(Nat.0)), x) = Real.0

        constant[Real, Real](Real.0, x) = Real.0
        pointwise_neg(pointwise_mul(constant[Real, Real](from_nat[Real](Nat.0)), sin_nt(Nat.0)), x) = constant[Real, Real](Real.0, x)

    }
    function_extensionality(pointwise_neg(pointwise_mul(constant[Real, Real](from_nat[Real](Nat.0)), sin_nt(Nat.0))), constant[Real, Real](Real.0))

    pointwise_neg(pointwise_mul(constant[Real, Real](from_nat[Real](Nat.0)), sin_nt(Nat.0))) = constant[Real, Real](Real.0)

    is_derivative_fn(cos_nt(Nat.0), pointwise_neg(pointwise_mul(constant[Real, Real](from_nat[Real](Nat.0)), sin_nt(Nat.0))))

    q(Nat.0)
    forall(m: Nat) {
        if q(m) {
            is_derivative_fn(sin_nt(m), pointwise_mul(constant[Real, Real](from_nat[Real](m)), cos_nt(m)))
            is_derivative_fn(cos_nt(m), pointwise_neg(pointwise_mul(constant[Real, Real](from_nat[Real](m)), sin_nt(m))))

            derivative_fn_mul(sin_nt(m), Real.cos, pointwise_mul(constant[Real, Real](from_nat[Real](m)), cos_nt(m)), pointwise_neg(Real.sin))

            is_derivative_fn(pointwise_mul(sin_nt(m), Real.cos), pointwise_add(pointwise_mul(sin_nt(m), pointwise_neg(Real.sin)), pointwise_mul(Real.cos, pointwise_mul(constant[Real, Real](from_nat[Real](m)), cos_nt(m)))))


            derivative_fn_mul(cos_nt(m), Real.sin, pointwise_neg(pointwise_mul(constant[Real, Real](from_nat[Real](m)), sin_nt(m))), Real.cos)

            is_derivative_fn(pointwise_mul(cos_nt(m), Real.sin), pointwise_add(pointwise_mul(cos_nt(m), Real.cos), pointwise_mul(Real.sin, pointwise_neg(pointwise_mul(constant[Real, Real](from_nat[Real](m)), sin_nt(m))))))


            derivative_fn_add(pointwise_mul(sin_nt(m), Real.cos), pointwise_mul(cos_nt(m), Real.sin), pointwise_add(pointwise_mul(sin_nt(m), pointwise_neg(Real.sin)), pointwise_mul(Real.cos, pointwise_mul(constant[Real, Real](from_nat[Real](m)), cos_nt(m)))), pointwise_add(pointwise_mul(cos_nt(m), Real.cos), pointwise_mul(Real.sin, pointwise_neg(pointwise_mul(constant[Real, Real](from_nat[Real](m)), sin_nt(m))))))




            forall(x: Real) {
                pointwise_mul_apply(sin_nt(m), Real.cos, x)
                pointwise_mul(sin_nt(m), Real.cos, x) = sin_nt(m, x) * x.cos
                pointwise_mul_apply(cos_nt(m), Real.sin, x)
                pointwise_mul(cos_nt(m), Real.sin, x) = cos_nt(m, x) * x.sin
                pointwise_add_apply(pointwise_mul(sin_nt(m), Real.cos), pointwise_mul(cos_nt(m), Real.sin), x)
                pointwise_add(pointwise_mul(sin_nt(m), Real.cos), pointwise_mul(cos_nt(m), Real.sin), x) = pointwise_mul(sin_nt(m), Real.cos, x) + pointwise_mul(cos_nt(m), Real.sin, x)

                pointwise_add(pointwise_mul(sin_nt(m), Real.cos), pointwise_mul(cos_nt(m), Real.sin), x) = sin_nt(m, x) * x.cos + cos_nt(m, x) * x.sin

                sin_nt_suc_decomp(m, x)
                sin_nt(m.suc, x) = sin_nt(m, x) * x.cos + cos_nt(m, x) * x.sin
                pointwise_add(pointwise_mul(sin_nt(m), Real.cos), pointwise_mul(cos_nt(m), Real.sin), x) = sin_nt(m.suc, x)

            }
            function_extensionality(pointwise_add(pointwise_mul(sin_nt(m), Real.cos), pointwise_mul(cos_nt(m), Real.sin)), sin_nt(m.suc))

            pointwise_add(pointwise_mul(sin_nt(m), Real.cos), pointwise_mul(cos_nt(m), Real.sin)) = sin_nt(m.suc)
            is_derivative_fn(sin_nt(m.suc), pointwise_add(

                    pointwise_add(pointwise_mul(sin_nt(m), pointwise_neg(Real.sin)), pointwise_mul(Real.cos, pointwise_mul(constant[Real, Real](from_nat[Real](m)), cos_nt(m)))), pointwise_add(pointwise_mul(cos_nt(m), Real.cos), pointwise_mul(Real.sin, pointwise_neg(pointwise_mul(constant[Real, Real](from_nat[Real](m)), sin_nt(m)))))))



            sin_deriv_expr_fn_eq(m)
            pointwise_add(pointwise_add(pointwise_mul(sin_nt(m), pointwise_neg(Real.sin)), pointwise_mul(Real.cos, pointwise_mul(constant[Real, Real](from_nat[Real](m)), cos_nt(m)))), pointwise_add(pointwise_mul(cos_nt(m), Real.cos), pointwise_mul(Real.sin, pointwise_neg(pointwise_mul(constant[Real, Real](from_nat[Real](m)), sin_nt(m)))))) = pointwise_mul(constant[Real, Real](from_nat[Real](m.suc)), cos_nt(m.suc))




            is_derivative_fn(sin_nt(m.suc), pointwise_mul(constant[Real, Real](from_nat[Real](m.suc)), cos_nt(m.suc)))

            derivative_fn_mul(cos_nt(m), Real.cos, pointwise_neg(pointwise_mul(constant[Real, Real](from_nat[Real](m)), sin_nt(m))), pointwise_neg(Real.sin))


            is_derivative_fn(pointwise_mul(cos_nt(m), Real.cos), pointwise_add(pointwise_mul(cos_nt(m), pointwise_neg(Real.sin)), pointwise_mul(Real.cos, pointwise_neg(pointwise_mul(constant[Real, Real](from_nat[Real](m)), sin_nt(m))))))


            derivative_fn_mul(sin_nt(m), Real.sin, pointwise_mul(constant[Real, Real](from_nat[Real](m)), cos_nt(m)), Real.cos)

            is_derivative_fn(pointwise_mul(sin_nt(m), Real.sin), pointwise_add(pointwise_mul(sin_nt(m), Real.cos), pointwise_mul(Real.sin, pointwise_mul(constant[Real, Real](from_nat[Real](m)), cos_nt(m)))))


            derivative_fn_neg(pointwise_mul(sin_nt(m), Real.sin), pointwise_add(pointwise_mul(sin_nt(m), Real.cos), pointwise_mul(Real.sin, pointwise_mul(constant[Real, Real](from_nat[Real](m)), cos_nt(m)))))


            is_derivative_fn(pointwise_neg(pointwise_mul(sin_nt(m), Real.sin)), pointwise_neg(pointwise_add(pointwise_mul(sin_nt(m), Real.cos), pointwise_mul(Real.sin, pointwise_mul(constant[Real, Real](from_nat[Real](m)), cos_nt(m))))))


            derivative_fn_add(pointwise_mul(cos_nt(m), Real.cos), pointwise_neg(pointwise_mul(sin_nt(m), Real.sin)), pointwise_add(pointwise_mul(cos_nt(m), pointwise_neg(Real.sin)), pointwise_mul(Real.cos, pointwise_neg(pointwise_mul(constant[Real, Real](from_nat[Real](m)), sin_nt(m))))), pointwise_neg(pointwise_add(pointwise_mul(sin_nt(m), Real.cos), pointwise_mul(Real.sin, pointwise_mul(constant[Real, Real](from_nat[Real](m)), cos_nt(m))))))




            forall(x: Real) {
                pointwise_mul_apply(cos_nt(m), Real.cos, x)
                pointwise_mul(cos_nt(m), Real.cos, x) = cos_nt(m, x) * x.cos
                pointwise_mul_apply(sin_nt(m), Real.sin, x)
                pointwise_mul(sin_nt(m), Real.sin, x) = sin_nt(m, x) * x.sin
                pointwise_neg_apply(pointwise_mul(sin_nt(m), Real.sin), x)
                pointwise_neg(pointwise_mul(sin_nt(m), Real.sin), x) = -(pointwise_mul(sin_nt(m), Real.sin, x))
                pointwise_neg(pointwise_mul(sin_nt(m), Real.sin), x) = -(sin_nt(m, x) * x.sin)
                pointwise_add_apply(pointwise_mul(cos_nt(m), Real.cos), pointwise_neg(pointwise_mul(sin_nt(m), Real.sin)), x)
                pointwise_add(pointwise_mul(cos_nt(m), Real.cos), pointwise_neg(pointwise_mul(sin_nt(m), Real.sin)), x) = pointwise_mul(cos_nt(m), Real.cos, x) + pointwise_neg(pointwise_mul(sin_nt(m), Real.sin), x)

                pointwise_add(pointwise_mul(cos_nt(m), Real.cos), pointwise_neg(pointwise_mul(sin_nt(m), Real.sin)), x) = cos_nt(m, x) * x.cos - sin_nt(m, x) * x.sin

                cos_nt_suc_decomp(m, x)
                cos_nt(m.suc, x) = cos_nt(m, x) * x.cos - sin_nt(m, x) * x.sin
                pointwise_add(pointwise_mul(cos_nt(m), Real.cos), pointwise_neg(pointwise_mul(sin_nt(m), Real.sin)), x) = cos_nt(m.suc, x)

            }
            function_extensionality(
                pointwise_add(pointwise_mul(cos_nt(m), Real.cos), pointwise_neg(pointwise_mul(sin_nt(m), Real.sin))), cos_nt(m.suc))

            pointwise_add(pointwise_mul(cos_nt(m), Real.cos), pointwise_neg(pointwise_mul(sin_nt(m), Real.sin))) = cos_nt(m.suc)

            is_derivative_fn(cos_nt(m.suc), pointwise_add(

                    pointwise_add(pointwise_mul(cos_nt(m), pointwise_neg(Real.sin)), pointwise_mul(Real.cos, pointwise_neg(pointwise_mul(constant[Real, Real](from_nat[Real](m)), sin_nt(m))))), pointwise_neg(pointwise_add(pointwise_mul(sin_nt(m), Real.cos), pointwise_mul(Real.sin, pointwise_mul(constant[Real, Real](from_nat[Real](m)), cos_nt(m)))))))



            cos_deriv_expr_fn_eq(m)
            pointwise_add(pointwise_add(pointwise_mul(cos_nt(m), pointwise_neg(Real.sin)), pointwise_mul(Real.cos, pointwise_neg(pointwise_mul(constant[Real, Real](from_nat[Real](m)), sin_nt(m))))), pointwise_neg(pointwise_add(pointwise_mul(sin_nt(m), Real.cos), pointwise_mul(Real.sin, pointwise_mul(constant[Real, Real](from_nat[Real](m)), cos_nt(m)))))) = pointwise_neg(pointwise_mul(constant[Real, Real](from_nat[Real](m.suc)), sin_nt(m.suc)))




            is_derivative_fn(cos_nt(m.suc), pointwise_neg(pointwise_mul(constant[Real, Real](from_nat[Real](m.suc)), sin_nt(m.suc))))

            q(m.suc)
        }
    }
    alt_induction(q)
    (q(Nat.0) and forall(j: Nat) { q(j) implies q(j.suc) }) implies forall(j: Nat) { q(j) }
    q(Nat.0) and forall(j: Nat) { q(j) implies q(j.suc) }
    forall(j: Nat) { q(j) }
    q(k)
    is_derivative_fn(cos_nt(k), pointwise_neg(pointwise_mul(constant[Real, Real](from_nat[Real](k)), sin_nt(k))))
}

// ---------------------------------------------------------------------------
// Lipschitz bounds and pointwise bounds of the harmonics
// ---------------------------------------------------------------------------

/// The real number from_nat(k) is nonnegative.
theorem from_nat_nonneg_real(k: Nat) {
    Real.0 <= from_nat[Real](k)
} by {
    define q(m: Nat) -> Bool {
        Real.0 <= from_nat[Real](m)
    }
    from_nat_zero[Real]
    from_nat[Real](Nat.0) = Real.0
    lte_self(Real.0)
    Real.0 <= Real.0
    Real.0 <= from_nat[Real](Nat.0)
    q(Nat.0)
    forall(m: Nat) {
        if q(m) {
            from_nat_add[Real](m, Nat.1)
            from_nat[Real](m + Nat.1) = from_nat[Real](m) + from_nat[Real](Nat.1)
            m + Nat.1 = m.suc
            from_nat[Real](m.suc) = from_nat[Real](m) + from_nat[Real](Nat.1)
            from_nat[Real](Nat.1) = Real.1
            from_nat[Real](m.suc) = from_nat[Real](m) + Real.1
            Real.0 <= from_nat[Real](m)
            one_nonneg
            Real.0 <= Real.1
            add_le_add(Real.0, from_nat[Real](m), Real.0, Real.1)
            Real.0 + Real.0 <= from_nat[Real](m) + Real.1
            Real.0 + Real.0 = Real.0
            Real.0 <= from_nat[Real](m) + Real.1
            Real.0 <= from_nat[Real](m.suc)
            q(m.suc)
        }
    }
    alt_induction(q)
    (q(Nat.0) and forall(j: Nat) { q(j) implies q(j.suc) }) implies forall(j: Nat) { q(j) }
    q(Nat.0) and forall(j: Nat) { q(j) implies q(j.suc) }
    forall(j: Nat) { q(j) }
    q(k)
}

/// The absolute value of from_nat(k) is from_nat(k).
theorem from_nat_abs(k: Nat) {
    (from_nat[Real](k)).abs = from_nat[Real](k)
} by {
    from_nat_nonneg_real(k)
    Real.0 <= from_nat[Real](k)
    abs_of_nonneg(from_nat[Real](k))
    from_nat[Real](k) >= Real.0 implies (from_nat[Real](k)).abs = from_nat[Real](k)
    (from_nat[Real](k)).abs = from_nat[Real](k)
}

/// The k-th sine harmonic is k-Lipschitz.
theorem sin_nt_lipschitz(k: Nat, u: Real, v: Real) {
    (sin_nt(k, u) - sin_nt(k, v)).abs <= from_nat[Real](k) * (u - v).abs
} by {
    sin_lipschitz(from_nat[Real](k) * u, from_nat[Real](k) * v)
    ((from_nat[Real](k) * u).sin - (from_nat[Real](k) * v).sin).abs <= (from_nat[Real](k) * u - from_nat[Real](k) * v).abs
    sin_nt(k, u) = (from_nat[Real](k) * u).sin
    sin_nt(k, v) = (from_nat[Real](k) * v).sin
    (from_nat[Real](k) * u).sin = sin_nt(k, u)
    (from_nat[Real](k) * v).sin = sin_nt(k, v)
    ((from_nat[Real](k) * u).sin - (from_nat[Real](k) * v).sin).abs = (sin_nt(k, u) - sin_nt(k, v)).abs

    (sin_nt(k, u) - sin_nt(k, v)).abs <= (from_nat[Real](k) * u - from_nat[Real](k) * v).abs
    mul_sub_distrib_left(from_nat[Real](k), u, v)
    from_nat[Real](k) * u - from_nat[Real](k) * v = from_nat[Real](k) * (u - v)
    (from_nat[Real](k) * u - from_nat[Real](k) * v).abs = (from_nat[Real](k) * (u - v)).abs
    mul_abs(from_nat[Real](k), u - v)
    (from_nat[Real](k) * (u - v)).abs = (from_nat[Real](k)).abs * (u - v).abs
    from_nat_abs(k)
    (from_nat[Real](k)).abs = from_nat[Real](k)
    (from_nat[Real](k) * (u - v)).abs = from_nat[Real](k) * (u - v).abs
    (from_nat[Real](k) * u - from_nat[Real](k) * v).abs = from_nat[Real](k) * (u - v).abs
    (sin_nt(k, u) - sin_nt(k, v)).abs <= from_nat[Real](k) * (u - v).abs
}

/// The k-th cosine harmonic is k-Lipschitz.
theorem cos_nt_lipschitz(k: Nat, u: Real, v: Real) {
    (cos_nt(k, u) - cos_nt(k, v)).abs <= from_nat[Real](k) * (u - v).abs
} by {
    cos_lipschitz(from_nat[Real](k) * u, from_nat[Real](k) * v)
    ((from_nat[Real](k) * u).cos - (from_nat[Real](k) * v).cos).abs <= (from_nat[Real](k) * u - from_nat[Real](k) * v).abs
    cos_nt(k, u) = (from_nat[Real](k) * u).cos
    cos_nt(k, v) = (from_nat[Real](k) * v).cos
    (from_nat[Real](k) * u).cos = cos_nt(k, u)
    (from_nat[Real](k) * v).cos = cos_nt(k, v)
    ((from_nat[Real](k) * u).cos - (from_nat[Real](k) * v).cos).abs = (cos_nt(k, u) - cos_nt(k, v)).abs

    (cos_nt(k, u) - cos_nt(k, v)).abs <= (from_nat[Real](k) * u - from_nat[Real](k) * v).abs
    mul_sub_distrib_left(from_nat[Real](k), u, v)
    from_nat[Real](k) * u - from_nat[Real](k) * v = from_nat[Real](k) * (u - v)
    (from_nat[Real](k) * u - from_nat[Real](k) * v).abs = (from_nat[Real](k) * (u - v)).abs
    mul_abs(from_nat[Real](k), u - v)
    (from_nat[Real](k) * (u - v)).abs = (from_nat[Real](k)).abs * (u - v).abs
    from_nat_abs(k)
    (from_nat[Real](k)).abs = from_nat[Real](k)
    (from_nat[Real](k) * (u - v)).abs = from_nat[Real](k) * (u - v).abs
    (from_nat[Real](k) * u - from_nat[Real](k) * v).abs = from_nat[Real](k) * (u - v).abs
    (cos_nt(k, u) - cos_nt(k, v)).abs <= from_nat[Real](k) * (u - v).abs
}

/// |kx.sin| <= 1.
theorem sin_nt_abs_le_one(k: Nat, x: Real) {
    (sin_nt(k, x)).abs <= Real.1
} by {
    sin_abs_le_one(from_nat[Real](k) * x)
    ((from_nat[Real](k) * x).sin).abs <= Real.1
    sin_nt(k, x) = (from_nat[Real](k) * x).sin
    (from_nat[Real](k) * x).sin = sin_nt(k, x)
    ((from_nat[Real](k) * x).sin).abs = (sin_nt(k, x)).abs
    (sin_nt(k, x)).abs <= Real.1
}

/// |kx.cos| <= 1.
theorem cos_nt_abs_le_one(k: Nat, x: Real) {
    (cos_nt(k, x)).abs <= Real.1
} by {
    cos_abs_le_one(from_nat[Real](k) * x)
    ((from_nat[Real](k) * x).cos).abs <= Real.1
    cos_nt(k, x) = (from_nat[Real](k) * x).cos
    (from_nat[Real](k) * x).cos = cos_nt(k, x)
    ((from_nat[Real](k) * x).cos).abs = (cos_nt(k, x)).abs
    (cos_nt(k, x)).abs <= Real.1
}

// ---------------------------------------------------------------------------
// Antiderivatives of the single harmonics, for k >= 1
// ---------------------------------------------------------------------------

/// The antiderivative of kx.sin on [0, 2*pi]: x -> -kx.cos/k.
define g_sin_nt(k: Nat) -> (Real -> Real) {
    pointwise_neg(pointwise_mul(constant[Real, Real](Real.1 / from_nat[Real](k)), cos_nt(k)))
}

/// The antiderivative of kx.cos on [0, 2*pi]: x -> kx.sin/k.
define g_cos_nt(k: Nat) -> (Real -> Real) {
    pointwise_mul(constant[Real, Real](Real.1 / from_nat[Real](k)), sin_nt(k))
}

/// From k >= 1 it follows that from_nat(k) is nonzero.
theorem from_nat_suc_ne_zero_from_pos(k: Nat) {
    Nat.0 < k implies from_nat[Real](k) != Real.0
} by {
    if Nat.0 < k {
        zero_or_suc(k)
        k = Nat.0 or exists(m: Nat) { k = m.suc }
        if k = Nat.0 {
            k = Nat.0
            Nat.0 < k
            Nat.0 < Nat.0
            lt_not_ref(Nat.0)
            not (Nat.0 < Nat.0)
            false
        }
        k != Nat.0
        exists(m: Nat) { k = m.suc }
        let m: Nat satisfy { k = m.suc }
        k = m.suc
        from_nat[Real](k) = from_nat[Real](m.suc)
        from_nat_suc_pos_real(m)
        from_nat[Real](m.suc) > Real.0
        Real.0 < from_nat[Real](m.suc)
        lt_imp_ne(Real.0, from_nat[Real](m.suc))
        Real.0 < from_nat[Real](m.suc) implies Real.0 != from_nat[Real](m.suc)
        Real.0 != from_nat[Real](m.suc)
        from_nat[Real](m.suc) != Real.0
        from_nat[Real](k) != Real.0
    }
}

/// The derivative of g_cos_nt(k) is cos_nt(k), for k >= 1.
theorem g_cos_nt_has_derivative(k: Nat) {
    Nat.0 < k implies is_derivative_fn(g_cos_nt(k), cos_nt(k))
} by {
    if Nat.0 < k {
        sin_nt_derivative(k)
        is_derivative_fn(sin_nt(k), pointwise_mul(constant[Real, Real](from_nat[Real](k)), cos_nt(k)))
        derivative_fn_const_mul(Real.1 / from_nat[Real](k), sin_nt(k), pointwise_mul(constant[Real, Real](from_nat[Real](k)), cos_nt(k)))

        is_derivative_fn(pointwise_mul(constant[Real, Real](Real.1 / from_nat[Real](k)), sin_nt(k)), pointwise_mul(constant[Real, Real](Real.1 / from_nat[Real](k)), pointwise_mul(constant[Real, Real](from_nat[Real](k)), cos_nt(k))))


        forall(x: Real) {
            pointwise_mul_apply(constant[Real, Real](Real.1 / from_nat[Real](k)), pointwise_mul(constant[Real, Real](from_nat[Real](k)), cos_nt(k)), x)

            pointwise_mul(constant[Real, Real](Real.1 / from_nat[Real](k)), pointwise_mul(constant[Real, Real](from_nat[Real](k)), cos_nt(k)), x) = constant[Real, Real](Real.1 / from_nat[Real](k), x) *


                    pointwise_mul(constant[Real, Real](from_nat[Real](k)), cos_nt(k), x)
            constant[Real, Real](Real.1 / from_nat[Real](k), x) = Real.1 / from_nat[Real](k)
            pointwise_mul_apply(constant[Real, Real](from_nat[Real](k)), cos_nt(k), x)
            pointwise_mul(constant[Real, Real](from_nat[Real](k)), cos_nt(k), x) = constant[Real, Real](from_nat[Real](k), x) * cos_nt(k, x)

            constant[Real, Real](from_nat[Real](k), x) = from_nat[Real](k)
            pointwise_mul(constant[Real, Real](from_nat[Real](k)), cos_nt(k), x) = from_nat[Real](k) * cos_nt(k, x)

            pointwise_mul(constant[Real, Real](Real.1 / from_nat[Real](k)), pointwise_mul(constant[Real, Real](from_nat[Real](k)), cos_nt(k)), x) = (Real.1 / from_nat[Real](k)) * (from_nat[Real](k) * cos_nt(k, x))


            from_nat_suc_ne_zero_from_pos(k)
            from_nat[Real](k) != Real.0
            mul_div_cancel(Real.1, from_nat[Real](k))
            from_nat[Real](k) != Real.0 implies from_nat[Real](k) * (Real.1 / from_nat[Real](k)) = Real.1
            from_nat[Real](k) * (Real.1 / from_nat[Real](k)) = Real.1
            (Real.1 / from_nat[Real](k)) * from_nat[Real](k) = Real.1
            mul_assoc(Real.1 / from_nat[Real](k), from_nat[Real](k), cos_nt(k, x))
            (Real.1 / from_nat[Real](k)) * (from_nat[Real](k) * cos_nt(k, x)) = ((Real.1 / from_nat[Real](k)) * from_nat[Real](k)) * cos_nt(k, x)

            ((Real.1 / from_nat[Real](k)) * from_nat[Real](k)) * cos_nt(k, x) = Real.1 * cos_nt(k, x)

            Real.1 * cos_nt(k, x) = cos_nt(k, x)
            (Real.1 / from_nat[Real](k)) * (from_nat[Real](k) * cos_nt(k, x)) = cos_nt(k, x)
            pointwise_mul(constant[Real, Real](Real.1 / from_nat[Real](k)), pointwise_mul(constant[Real, Real](from_nat[Real](k)), cos_nt(k)), x) = cos_nt(k, x)

        }
        function_extensionality(pointwise_mul(constant[Real, Real](Real.1 / from_nat[Real](k)), pointwise_mul(constant[Real, Real](from_nat[Real](k)), cos_nt(k))), cos_nt(k))

        pointwise_mul(constant[Real, Real](Real.1 / from_nat[Real](k)), pointwise_mul(constant[Real, Real](from_nat[Real](k)), cos_nt(k))) = cos_nt(k)

        is_derivative_fn(pointwise_mul(constant[Real, Real](Real.1 / from_nat[Real](k)), sin_nt(k)), cos_nt(k))

        g_cos_nt(k) = pointwise_mul(constant[Real, Real](Real.1 / from_nat[Real](k)), sin_nt(k))
        is_derivative_fn(g_cos_nt(k), cos_nt(k))
    }
}

/// The derivative of g_sin_nt(k) is sin_nt(k), for k >= 1.
theorem g_sin_nt_has_derivative(k: Nat) {
    Nat.0 < k implies is_derivative_fn(g_sin_nt(k), sin_nt(k))
} by {
    if Nat.0 < k {
        cos_nt_derivative(k)
        is_derivative_fn(cos_nt(k), pointwise_neg(pointwise_mul(constant[Real, Real](from_nat[Real](k)), sin_nt(k))))
        derivative_fn_const_mul(Real.1 / from_nat[Real](k), cos_nt(k), pointwise_neg(pointwise_mul(constant[Real, Real](from_nat[Real](k)), sin_nt(k))))

        is_derivative_fn(pointwise_mul(constant[Real, Real](Real.1 / from_nat[Real](k)), cos_nt(k)), pointwise_mul(constant[Real, Real](Real.1 / from_nat[Real](k)), pointwise_neg(pointwise_mul(constant[Real, Real](from_nat[Real](k)), sin_nt(k)))))


        derivative_fn_neg(pointwise_mul(constant[Real, Real](Real.1 / from_nat[Real](k)), cos_nt(k)), pointwise_mul(constant[Real, Real](Real.1 / from_nat[Real](k)), pointwise_neg(pointwise_mul(constant[Real, Real](from_nat[Real](k)), sin_nt(k)))))


        is_derivative_fn(pointwise_neg(pointwise_mul(constant[Real, Real](Real.1 / from_nat[Real](k)), cos_nt(k))), pointwise_neg(pointwise_mul(constant[Real, Real](Real.1 / from_nat[Real](k)), pointwise_neg(pointwise_mul(constant[Real, Real](from_nat[Real](k)), sin_nt(k))))))


        forall(x: Real) {
            pointwise_neg_apply(pointwise_mul(constant[Real, Real](Real.1 / from_nat[Real](k)), pointwise_neg(pointwise_mul(constant[Real, Real](from_nat[Real](k)), sin_nt(k)))), x)

            pointwise_neg(pointwise_mul(constant[Real, Real](Real.1 / from_nat[Real](k)), pointwise_neg(pointwise_mul(constant[Real, Real](from_nat[Real](k)), sin_nt(k)))), x) = -(pointwise_mul(constant[Real, Real](Real.1 / from_nat[Real](k)), pointwise_neg(pointwise_mul(constant[Real, Real](from_nat[Real](k)), sin_nt(k))), x))


            pointwise_mul_apply(constant[Real, Real](Real.1 / from_nat[Real](k)), pointwise_neg(pointwise_mul(constant[Real, Real](from_nat[Real](k)), sin_nt(k))), x)

            pointwise_mul(constant[Real, Real](Real.1 / from_nat[Real](k)), pointwise_neg(pointwise_mul(constant[Real, Real](from_nat[Real](k)), sin_nt(k))), x) = constant[Real, Real](Real.1 / from_nat[Real](k), x) *


                    pointwise_neg(pointwise_mul(constant[Real, Real](from_nat[Real](k)), sin_nt(k)), x)
            constant[Real, Real](Real.1 / from_nat[Real](k), x) = Real.1 / from_nat[Real](k)
            pointwise_neg_apply(pointwise_mul(constant[Real, Real](from_nat[Real](k)), sin_nt(k)), x)
            pointwise_neg(pointwise_mul(constant[Real, Real](from_nat[Real](k)), sin_nt(k)), x) = -(pointwise_mul(constant[Real, Real](from_nat[Real](k)), sin_nt(k), x))

            pointwise_mul_apply(constant[Real, Real](from_nat[Real](k)), sin_nt(k), x)
            pointwise_mul(constant[Real, Real](from_nat[Real](k)), sin_nt(k), x) = constant[Real, Real](from_nat[Real](k), x) * sin_nt(k, x)

            constant[Real, Real](from_nat[Real](k), x) = from_nat[Real](k)
            pointwise_mul(constant[Real, Real](from_nat[Real](k)), sin_nt(k), x) = from_nat[Real](k) * sin_nt(k, x)

            pointwise_neg(pointwise_mul(constant[Real, Real](from_nat[Real](k)), sin_nt(k)), x) = -(from_nat[Real](k) * sin_nt(k, x))

            pointwise_mul(constant[Real, Real](Real.1 / from_nat[Real](k)), pointwise_neg(pointwise_mul(constant[Real, Real](from_nat[Real](k)), sin_nt(k))), x) = (Real.1 / from_nat[Real](k)) * (-(from_nat[Real](k) * sin_nt(k, x)))


            pointwise_neg(pointwise_mul(constant[Real, Real](Real.1 / from_nat[Real](k)), pointwise_neg(pointwise_mul(constant[Real, Real](from_nat[Real](k)), sin_nt(k)))), x) = -((Real.1 / from_nat[Real](k)) * (-(from_nat[Real](k) * sin_nt(k, x))))


            from_nat_suc_ne_zero_from_pos(k)
            from_nat[Real](k) != Real.0
            mul_div_cancel(Real.1, from_nat[Real](k))
            from_nat[Real](k) != Real.0 implies from_nat[Real](k) * (Real.1 / from_nat[Real](k)) = Real.1
            from_nat[Real](k) * (Real.1 / from_nat[Real](k)) = Real.1
            (Real.1 / from_nat[Real](k)) * from_nat[Real](k) = Real.1
            mul_assoc(Real.1 / from_nat[Real](k), from_nat[Real](k), sin_nt(k, x))
            (Real.1 / from_nat[Real](k)) * (from_nat[Real](k) * sin_nt(k, x)) = ((Real.1 / from_nat[Real](k)) * from_nat[Real](k)) * sin_nt(k, x)

            ((Real.1 / from_nat[Real](k)) * from_nat[Real](k)) * sin_nt(k, x) = Real.1 * sin_nt(k, x)

            Real.1 * sin_nt(k, x) = sin_nt(k, x)
            (Real.1 / from_nat[Real](k)) * (from_nat[Real](k) * sin_nt(k, x)) = sin_nt(k, x)
            (Real.1 / from_nat[Real](k)) * (-(from_nat[Real](k) * sin_nt(k, x))) = -((Real.1 / from_nat[Real](k)) * (from_nat[Real](k) * sin_nt(k, x)))

            -((Real.1 / from_nat[Real](k)) * (-(from_nat[Real](k) * sin_nt(k, x)))) = sin_nt(k, x)
            pointwise_neg(pointwise_mul(constant[Real, Real](Real.1 / from_nat[Real](k)), pointwise_neg(pointwise_mul(constant[Real, Real](from_nat[Real](k)), sin_nt(k)))), x) = sin_nt(k, x)


        }
        function_extensionality(pointwise_neg(pointwise_mul(constant[Real, Real](Real.1 / from_nat[Real](k)), pointwise_neg(pointwise_mul(constant[Real, Real](from_nat[Real](k)), sin_nt(k))))), sin_nt(k))

        pointwise_neg(pointwise_mul(constant[Real, Real](Real.1 / from_nat[Real](k)), pointwise_neg(pointwise_mul(constant[Real, Real](from_nat[Real](k)), sin_nt(k))))) = sin_nt(k)

        is_derivative_fn(pointwise_neg(pointwise_mul(constant[Real, Real](Real.1 / from_nat[Real](k)), cos_nt(k))), sin_nt(k))

        g_sin_nt(k) = pointwise_neg(pointwise_mul(constant[Real, Real](Real.1 / from_nat[Real](k)), cos_nt(k)))
        is_derivative_fn(g_sin_nt(k), sin_nt(k))
    }
}

/// The antiderivative g_cos_nt(k) is continuous.
theorem g_cos_nt_continuous(k: Nat) {
    continuous(g_cos_nt(k))
} by {
    sin_nt_continuous(k)
    continuous(sin_nt(k))
    continuous_const_mul_left(Real.1 / from_nat[Real](k), sin_nt(k))
    continuous(sin_nt(k)) implies continuous(const_mul_left(Real.1 / from_nat[Real](k), sin_nt(k)))
    continuous(const_mul_left(Real.1 / from_nat[Real](k), sin_nt(k)))
    forall(x: Real) {
        const_mul_left(Real.1 / from_nat[Real](k), sin_nt(k), x) = (Real.1 / from_nat[Real](k)) * sin_nt(k, x)

        pointwise_mul_apply(constant[Real, Real](Real.1 / from_nat[Real](k)), sin_nt(k), x)
        pointwise_mul(constant[Real, Real](Real.1 / from_nat[Real](k)), sin_nt(k), x) = constant[Real, Real](Real.1 / from_nat[Real](k), x) * sin_nt(k, x)

        constant[Real, Real](Real.1 / from_nat[Real](k), x) = Real.1 / from_nat[Real](k)
        pointwise_mul(constant[Real, Real](Real.1 / from_nat[Real](k)), sin_nt(k), x) = (Real.1 / from_nat[Real](k)) * sin_nt(k, x)

        const_mul_left(Real.1 / from_nat[Real](k), sin_nt(k), x) = pointwise_mul(constant[Real, Real](Real.1 / from_nat[Real](k)), sin_nt(k), x)

    }
    function_extensionality(const_mul_left(Real.1 / from_nat[Real](k), sin_nt(k)), pointwise_mul(constant[Real, Real](Real.1 / from_nat[Real](k)), sin_nt(k)))

    const_mul_left(Real.1 / from_nat[Real](k), sin_nt(k)) = pointwise_mul(constant[Real, Real](Real.1 / from_nat[Real](k)), sin_nt(k))

    continuous(pointwise_mul(constant[Real, Real](Real.1 / from_nat[Real](k)), sin_nt(k)))
    continuous(g_cos_nt(k))
}

/// The antiderivative g_sin_nt(k) is continuous.
theorem g_sin_nt_continuous(k: Nat) {
    continuous(g_sin_nt(k))
} by {
    cos_nt_continuous(k)
    continuous(cos_nt(k))
    continuous_pointwise_neg(cos_nt(k))
    continuous(pointwise_neg(cos_nt(k)))
    continuous_const_mul_left(Real.1 / from_nat[Real](k), pointwise_neg(cos_nt(k)))
    continuous(pointwise_neg(cos_nt(k))) implies continuous(const_mul_left(Real.1 / from_nat[Real](k), pointwise_neg(cos_nt(k))))
    continuous(const_mul_left(Real.1 / from_nat[Real](k), pointwise_neg(cos_nt(k))))
    forall(x: Real) {
        const_mul_left(Real.1 / from_nat[Real](k), pointwise_neg(cos_nt(k)), x) = (Real.1 / from_nat[Real](k)) * pointwise_neg(cos_nt(k), x)

        pointwise_neg_apply(cos_nt(k), x)
        pointwise_neg(cos_nt(k), x) = -cos_nt(k, x)
        pointwise_mul_apply(constant[Real, Real](Real.1 / from_nat[Real](k)), pointwise_neg(cos_nt(k)), x)
        pointwise_mul(constant[Real, Real](Real.1 / from_nat[Real](k)), pointwise_neg(cos_nt(k)), x) = constant[Real, Real](Real.1 / from_nat[Real](k), x) * pointwise_neg(cos_nt(k), x)

        constant[Real, Real](Real.1 / from_nat[Real](k), x) = Real.1 / from_nat[Real](k)
        pointwise_mul(constant[Real, Real](Real.1 / from_nat[Real](k)), pointwise_neg(cos_nt(k)), x) = (Real.1 / from_nat[Real](k)) * (-cos_nt(k, x))

        const_mul_left(Real.1 / from_nat[Real](k), pointwise_neg(cos_nt(k)), x) = pointwise_mul(constant[Real, Real](Real.1 / from_nat[Real](k)), pointwise_neg(cos_nt(k)), x)

    }
    function_extensionality(const_mul_left(Real.1 / from_nat[Real](k), pointwise_neg(cos_nt(k))), pointwise_mul(constant[Real, Real](Real.1 / from_nat[Real](k)), pointwise_neg(cos_nt(k))))

    const_mul_left(Real.1 / from_nat[Real](k), pointwise_neg(cos_nt(k))) = pointwise_mul(constant[Real, Real](Real.1 / from_nat[Real](k)), pointwise_neg(cos_nt(k)))

    continuous(pointwise_mul(constant[Real, Real](Real.1 / from_nat[Real](k)), pointwise_neg(cos_nt(k))))
    forall(x: Real) {
        pointwise_mul_apply(constant[Real, Real](Real.1 / from_nat[Real](k)), pointwise_neg(cos_nt(k)), x)
        pointwise_mul(constant[Real, Real](Real.1 / from_nat[Real](k)), pointwise_neg(cos_nt(k)), x) = constant[Real, Real](Real.1 / from_nat[Real](k), x) * pointwise_neg(cos_nt(k), x)

        constant[Real, Real](Real.1 / from_nat[Real](k), x) = Real.1 / from_nat[Real](k)
        pointwise_neg_apply(cos_nt(k), x)
        pointwise_neg(cos_nt(k), x) = -cos_nt(k, x)
        pointwise_mul(constant[Real, Real](Real.1 / from_nat[Real](k)), pointwise_neg(cos_nt(k)), x) = (Real.1 / from_nat[Real](k)) * (-cos_nt(k, x))

        mul_neg_right(Real.1 / from_nat[Real](k), cos_nt(k, x))
        (Real.1 / from_nat[Real](k)) * (-cos_nt(k, x)) = -((Real.1 / from_nat[Real](k)) * cos_nt(k, x))
        pointwise_mul_apply(constant[Real, Real](Real.1 / from_nat[Real](k)), cos_nt(k), x)
        pointwise_mul(constant[Real, Real](Real.1 / from_nat[Real](k)), cos_nt(k), x) = constant[Real, Real](Real.1 / from_nat[Real](k), x) * cos_nt(k, x)

        constant[Real, Real](Real.1 / from_nat[Real](k), x) = Real.1 / from_nat[Real](k)
        pointwise_mul(constant[Real, Real](Real.1 / from_nat[Real](k)), cos_nt(k), x) = (Real.1 / from_nat[Real](k)) * cos_nt(k, x)

        pointwise_neg_apply(pointwise_mul(constant[Real, Real](Real.1 / from_nat[Real](k)), cos_nt(k)), x)
        pointwise_neg(pointwise_mul(constant[Real, Real](Real.1 / from_nat[Real](k)), cos_nt(k)), x) = -(pointwise_mul(constant[Real, Real](Real.1 / from_nat[Real](k)), cos_nt(k), x))

        pointwise_neg(pointwise_mul(constant[Real, Real](Real.1 / from_nat[Real](k)), cos_nt(k)), x) = -((Real.1 / from_nat[Real](k)) * cos_nt(k, x))

        pointwise_mul(constant[Real, Real](Real.1 / from_nat[Real](k)), pointwise_neg(cos_nt(k)), x) = pointwise_neg(pointwise_mul(constant[Real, Real](Real.1 / from_nat[Real](k)), cos_nt(k)), x)

    }
    function_extensionality(pointwise_mul(constant[Real, Real](Real.1 / from_nat[Real](k)), pointwise_neg(cos_nt(k))), pointwise_neg(pointwise_mul(constant[Real, Real](Real.1 / from_nat[Real](k)), cos_nt(k))))

    pointwise_mul(constant[Real, Real](Real.1 / from_nat[Real](k)), pointwise_neg(cos_nt(k))) = pointwise_neg(pointwise_mul(constant[Real, Real](Real.1 / from_nat[Real](k)), cos_nt(k)))

    continuous(pointwise_neg(pointwise_mul(constant[Real, Real](Real.1 / from_nat[Real](k)), cos_nt(k))))
    g_sin_nt(k) = pointwise_neg(pointwise_mul(constant[Real, Real](Real.1 / from_nat[Real](k)), cos_nt(k)))
    continuous(g_sin_nt(k))
}

// ---------------------------------------------------------------------------
// The integrals of the single harmonics over one full period
// ---------------------------------------------------------------------------

/// kx.sin is integrable on [0, 2*pi] for every k >= 1.
theorem sin_nt_integrable(k: Nat) {
    is_integrable(sin_nt(k.suc), Real.0, two_pi)
} by {
        two_pi_nonneg
        Real.0 <= two_pi
        from_nat_nonneg_real(k.suc)
        Real.0 <= from_nat[Real](k.suc)
        g_sin_nt_continuous(k.suc)
        continuous(g_sin_nt(k.suc))
        g_sin_nt_has_derivative(k.suc)
        lt_add_suc(Nat.0, k)
        Nat.0 < Nat.0 + k.suc
        Nat.0 + k.suc = k.suc
        Nat.0 < k.suc
        Nat.0 < k.suc implies is_derivative_fn(g_sin_nt(k.suc), sin_nt(k.suc))
        is_derivative_fn(g_sin_nt(k.suc), sin_nt(k.suc))
        forall(u: Real, v: Real) {
            if interval_contains(Real.0, two_pi, u) and interval_contains(Real.0, two_pi, v) {
                sin_nt_lipschitz(k.suc, u, v)
                (sin_nt(k.suc, u) - sin_nt(k.suc, v)).abs <= from_nat[Real](k.suc) * (u - v).abs
            }
        }
        forall(t: Real) {
            if interval_contains(Real.0, two_pi, t) {
                sin_nt_abs_le_one(k.suc, t)
                (sin_nt(k.suc, t)).abs <= Real.1
                abs_le_one_imp_lower(sin_nt(k.suc, t))
                -Real.1 <= sin_nt(k.suc, t)
            }
        }
        forall(t: Real) {
            if interval_contains(Real.0, two_pi, t) {
                sin_nt_abs_le_one(k.suc, t)
                (sin_nt(k.suc, t)).abs <= Real.1
                abs_le_one_imp_upper(sin_nt(k.suc, t))
                sin_nt(k.suc, t) <= Real.1
            }
        }
        Real.0 <= two_pi
        (Real.0 <= two_pi) and Real.0 <= from_nat[Real](k.suc)
        ((Real.0 <= two_pi) and Real.0 <= from_nat[Real](k.suc)) and continuous(g_sin_nt(k.suc))
        (((Real.0 <= two_pi) and Real.0 <= from_nat[Real](k.suc)) and continuous(g_sin_nt(k.suc))) and is_derivative_fn(g_sin_nt(k.suc), sin_nt(k.suc))

        ((((Real.0 <= two_pi) and Real.0 <= from_nat[Real](k.suc)) and continuous(g_sin_nt(k.suc))) and is_derivative_fn(g_sin_nt(k.suc), sin_nt(k.suc))) and forall(u: Real, v: Real) {

                interval_contains(Real.0, two_pi, u) and interval_contains(Real.0, two_pi, v)
                implies (sin_nt(k.suc, u) - sin_nt(k.suc, v)).abs <= from_nat[Real](k.suc) * (u - v).abs
            }
        (((((Real.0 <= two_pi) and Real.0 <= from_nat[Real](k.suc)) and continuous(g_sin_nt(k.suc))) and is_derivative_fn(g_sin_nt(k.suc), sin_nt(k.suc))) and forall(u: Real, v: Real) {

                interval_contains(Real.0, two_pi, u) and interval_contains(Real.0, two_pi, v)
                implies (sin_nt(k.suc, u) - sin_nt(k.suc, v)).abs <= from_nat[Real](k.suc) * (u - v).abs
            }) and forall(t: Real) { interval_contains(Real.0, two_pi, t) implies -Real.1 <= sin_nt(k.suc, t) }
        ((((((Real.0 <= two_pi) and Real.0 <= from_nat[Real](k.suc)) and continuous(g_sin_nt(k.suc))) and is_derivative_fn(g_sin_nt(k.suc), sin_nt(k.suc))) and forall(u: Real, v: Real) {

                interval_contains(Real.0, two_pi, u) and interval_contains(Real.0, two_pi, v)
                implies (sin_nt(k.suc, u) - sin_nt(k.suc, v)).abs <= from_nat[Real](k.suc) * (u - v).abs
            }) and forall(t: Real) { interval_contains(Real.0, two_pi, t) implies -Real.1 <= sin_nt(k.suc, t) }) and forall(t: Real) { interval_contains(Real.0, two_pi, t) implies sin_nt(k.suc, t) <= Real.1 }

        if ((((((Real.0 <= two_pi) and Real.0 <= from_nat[Real](k.suc)) and continuous(g_sin_nt(k.suc))) and is_derivative_fn(g_sin_nt(k.suc), sin_nt(k.suc))) and forall(u: Real, v: Real) {

                interval_contains(Real.0, two_pi, u) and interval_contains(Real.0, two_pi, v)
                implies (sin_nt(k.suc, u) - sin_nt(k.suc, v)).abs <= from_nat[Real](k.suc) * (u - v).abs
            }) and forall(t: Real) { interval_contains(Real.0, two_pi, t) implies -Real.1 <= sin_nt(k.suc, t) }) and forall(t: Real) { interval_contains(Real.0, two_pi, t) implies sin_nt(k.suc, t) <= Real.1 } {

            fn_integrable_gen(sin_nt(k.suc), g_sin_nt(k.suc), Real.0, two_pi, from_nat[Real](k.suc), -Real.1, Real.1)
            is_integrable(sin_nt(k.suc), Real.0, two_pi)
        }
}

/// The integral of kx.sin over [0, 2*pi] is zero for every k >= 1.
theorem integral_sin_nt_zero(k: Nat) {
    integral(sin_nt(k.suc), Real.0, two_pi) = Real.0
} by {
        two_pi_nonneg
        Real.0 <= two_pi
        g_sin_nt_continuous(k.suc)
        continuous(g_sin_nt(k.suc))
        g_sin_nt_has_derivative(k.suc)
        lt_add_suc(Nat.0, k)
        Nat.0 < Nat.0 + k.suc
        Nat.0 + k.suc = k.suc
        Nat.0 < k.suc
        Nat.0 < k.suc implies is_derivative_fn(g_sin_nt(k.suc), sin_nt(k.suc))
        is_derivative_fn(g_sin_nt(k.suc), sin_nt(k.suc))
        sin_nt_integrable(k)
        is_integrable(sin_nt(k.suc), Real.0, two_pi)
        forall(t: Real) {
            if interval_contains(Real.0, two_pi, t) {
                sin_nt_abs_le_one(k.suc, t)
                (sin_nt(k.suc, t)).abs <= Real.1
                abs_le_one_imp_lower(sin_nt(k.suc, t))
                -Real.1 <= sin_nt(k.suc, t)
            }
        }
        forall(t: Real) {
            if interval_contains(Real.0, two_pi, t) {
                sin_nt_abs_le_one(k.suc, t)
                (sin_nt(k.suc, t)).abs <= Real.1
                abs_le_one_imp_upper(sin_nt(k.suc, t))
                sin_nt(k.suc, t) <= Real.1
            }
        }
        Real.0 <= two_pi
        (Real.0 <= two_pi) and continuous(g_sin_nt(k.suc))
        ((Real.0 <= two_pi) and continuous(g_sin_nt(k.suc))) and is_derivative_fn(g_sin_nt(k.suc), sin_nt(k.suc))
        (((Real.0 <= two_pi) and continuous(g_sin_nt(k.suc))) and is_derivative_fn(g_sin_nt(k.suc), sin_nt(k.suc))) and is_integrable(sin_nt(k.suc), Real.0, two_pi)

        ((((Real.0 <= two_pi) and continuous(g_sin_nt(k.suc))) and is_derivative_fn(g_sin_nt(k.suc), sin_nt(k.suc))) and is_integrable(sin_nt(k.suc), Real.0, two_pi)) and forall(t: Real) {

                interval_contains(Real.0, two_pi, t) implies -Real.1 <= sin_nt(k.suc, t)
            }
        (((((Real.0 <= two_pi) and continuous(g_sin_nt(k.suc))) and is_derivative_fn(g_sin_nt(k.suc), sin_nt(k.suc))) and is_integrable(sin_nt(k.suc), Real.0, two_pi)) and forall(t: Real) {

                interval_contains(Real.0, two_pi, t) implies -Real.1 <= sin_nt(k.suc, t)
            }) and forall(t: Real) { interval_contains(Real.0, two_pi, t) implies sin_nt(k.suc, t) <= Real.1 }
        if (((((Real.0 <= two_pi) and continuous(g_sin_nt(k.suc))) and is_derivative_fn(g_sin_nt(k.suc), sin_nt(k.suc))) and is_integrable(sin_nt(k.suc), Real.0, two_pi)) and forall(t: Real) {

                interval_contains(Real.0, two_pi, t) implies -Real.1 <= sin_nt(k.suc, t)
            }) and forall(t: Real) { interval_contains(Real.0, two_pi, t) implies sin_nt(k.suc, t) <= Real.1 } {
            ftc2_general(sin_nt(k.suc), g_sin_nt(k.suc), Real.0, two_pi, -Real.1, Real.1)
            integral(sin_nt(k.suc), Real.0, two_pi) = g_sin_nt(k.suc)(two_pi) - g_sin_nt(k.suc)(Real.0)
            pointwise_mul_apply(constant[Real, Real](Real.1 / from_nat[Real](k.suc)), cos_nt(k.suc), two_pi)
            pointwise_mul(constant[Real, Real](Real.1 / from_nat[Real](k.suc)), cos_nt(k.suc), two_pi) = constant[Real, Real](Real.1 / from_nat[Real](k.suc), two_pi) * cos_nt(k.suc, two_pi)

            constant[Real, Real](Real.1 / from_nat[Real](k.suc), two_pi) = Real.1 / from_nat[Real](k.suc)
            pointwise_mul(constant[Real, Real](Real.1 / from_nat[Real](k.suc)), cos_nt(k.suc), two_pi) = (Real.1 / from_nat[Real](k.suc)) * cos_nt(k.suc, two_pi)

            pointwise_neg_apply(pointwise_mul(constant[Real, Real](Real.1 / from_nat[Real](k.suc)), cos_nt(k.suc)), two_pi)
            pointwise_neg(pointwise_mul(constant[Real, Real](Real.1 / from_nat[Real](k.suc)), cos_nt(k.suc)), two_pi) = -(pointwise_mul(constant[Real, Real](Real.1 / from_nat[Real](k.suc)), cos_nt(k.suc), two_pi))

            pointwise_neg(pointwise_mul(constant[Real, Real](Real.1 / from_nat[Real](k.suc)), cos_nt(k.suc)), two_pi) = -((Real.1 / from_nat[Real](k.suc)) * cos_nt(k.suc, two_pi))

            g_sin_nt(k.suc)(two_pi) = -((Real.1 / from_nat[Real](k.suc)) * cos_nt(k.suc, two_pi))
            cos_nt_two_pi_one(k.suc)
            cos_nt(k.suc, two_pi) = Real.1
            (Real.1 / from_nat[Real](k.suc)) * cos_nt(k.suc, two_pi) = Real.1 / from_nat[Real](k.suc)
            g_sin_nt(k.suc)(two_pi) = -(Real.1 / from_nat[Real](k.suc))
            pointwise_mul_apply(constant[Real, Real](Real.1 / from_nat[Real](k.suc)), cos_nt(k.suc), Real.0)
            pointwise_mul(constant[Real, Real](Real.1 / from_nat[Real](k.suc)), cos_nt(k.suc), Real.0) = constant[Real, Real](Real.1 / from_nat[Real](k.suc), Real.0) * cos_nt(k.suc, Real.0)

            constant[Real, Real](Real.1 / from_nat[Real](k.suc), Real.0) = Real.1 / from_nat[Real](k.suc)
            pointwise_mul(constant[Real, Real](Real.1 / from_nat[Real](k.suc)), cos_nt(k.suc), Real.0) = (Real.1 / from_nat[Real](k.suc)) * cos_nt(k.suc, Real.0)

            pointwise_neg_apply(pointwise_mul(constant[Real, Real](Real.1 / from_nat[Real](k.suc)), cos_nt(k.suc)), Real.0)
            pointwise_neg(pointwise_mul(constant[Real, Real](Real.1 / from_nat[Real](k.suc)), cos_nt(k.suc)), Real.0) = -(pointwise_mul(constant[Real, Real](Real.1 / from_nat[Real](k.suc)), cos_nt(k.suc), Real.0))

            pointwise_neg(pointwise_mul(constant[Real, Real](Real.1 / from_nat[Real](k.suc)), cos_nt(k.suc)), Real.0) = -((Real.1 / from_nat[Real](k.suc)) * cos_nt(k.suc, Real.0))

            g_sin_nt(k.suc)(Real.0) = -((Real.1 / from_nat[Real](k.suc)) * cos_nt(k.suc, Real.0))
            cos_nt_at_zero(k.suc)
            cos_nt(k.suc, Real.0) = Real.1
            (Real.1 / from_nat[Real](k.suc)) * cos_nt(k.suc, Real.0) = Real.1 / from_nat[Real](k.suc)
            g_sin_nt(k.suc)(Real.0) = -(Real.1 / from_nat[Real](k.suc))
            g_sin_nt(k.suc)(two_pi) - g_sin_nt(k.suc)(Real.0) = -(Real.1 / from_nat[Real](k.suc)) - (-(Real.1 / from_nat[Real](k.suc)))
            -(Real.1 / from_nat[Real](k.suc)) - (-(Real.1 / from_nat[Real](k.suc))) = Real.0
            integral(sin_nt(k.suc), Real.0, two_pi) = Real.0
        }
}

/// kx.cos is integrable on [0, 2*pi] for every k >= 1.
theorem cos_nt_integrable(k: Nat) {
    is_integrable(cos_nt(k.suc), Real.0, two_pi)
} by {
        two_pi_nonneg
        Real.0 <= two_pi
        from_nat_nonneg_real(k.suc)
        Real.0 <= from_nat[Real](k.suc)
        g_cos_nt_continuous(k.suc)
        continuous(g_cos_nt(k.suc))
        g_cos_nt_has_derivative(k.suc)
        lt_add_suc(Nat.0, k)
        Nat.0 < Nat.0 + k.suc
        Nat.0 + k.suc = k.suc
        Nat.0 < k.suc
        Nat.0 < k.suc implies is_derivative_fn(g_cos_nt(k.suc), cos_nt(k.suc))
        is_derivative_fn(g_cos_nt(k.suc), cos_nt(k.suc))
        forall(u: Real, v: Real) {
            if interval_contains(Real.0, two_pi, u) and interval_contains(Real.0, two_pi, v) {
                cos_nt_lipschitz(k.suc, u, v)
                (cos_nt(k.suc, u) - cos_nt(k.suc, v)).abs <= from_nat[Real](k.suc) * (u - v).abs
            }
        }
        forall(t: Real) {
            if interval_contains(Real.0, two_pi, t) {
                cos_nt_abs_le_one(k.suc, t)
                (cos_nt(k.suc, t)).abs <= Real.1
                abs_le_one_imp_lower(cos_nt(k.suc, t))
                -Real.1 <= cos_nt(k.suc, t)
            }
        }
        forall(t: Real) {
            if interval_contains(Real.0, two_pi, t) {
                cos_nt_abs_le_one(k.suc, t)
                (cos_nt(k.suc, t)).abs <= Real.1
                abs_le_one_imp_upper(cos_nt(k.suc, t))
                cos_nt(k.suc, t) <= Real.1
            }
        }
        Real.0 <= two_pi
        (Real.0 <= two_pi) and Real.0 <= from_nat[Real](k.suc)
        ((Real.0 <= two_pi) and Real.0 <= from_nat[Real](k.suc)) and continuous(g_cos_nt(k.suc))
        (((Real.0 <= two_pi) and Real.0 <= from_nat[Real](k.suc)) and continuous(g_cos_nt(k.suc))) and is_derivative_fn(g_cos_nt(k.suc), cos_nt(k.suc))

        ((((Real.0 <= two_pi) and Real.0 <= from_nat[Real](k.suc)) and continuous(g_cos_nt(k.suc))) and is_derivative_fn(g_cos_nt(k.suc), cos_nt(k.suc))) and forall(u: Real, v: Real) {

                interval_contains(Real.0, two_pi, u) and interval_contains(Real.0, two_pi, v)
                implies (cos_nt(k.suc, u) - cos_nt(k.suc, v)).abs <= from_nat[Real](k.suc) * (u - v).abs
            }
        (((((Real.0 <= two_pi) and Real.0 <= from_nat[Real](k.suc)) and continuous(g_cos_nt(k.suc))) and is_derivative_fn(g_cos_nt(k.suc), cos_nt(k.suc))) and forall(u: Real, v: Real) {

                interval_contains(Real.0, two_pi, u) and interval_contains(Real.0, two_pi, v)
                implies (cos_nt(k.suc, u) - cos_nt(k.suc, v)).abs <= from_nat[Real](k.suc) * (u - v).abs
            }) and forall(t: Real) { interval_contains(Real.0, two_pi, t) implies -Real.1 <= cos_nt(k.suc, t) }
        ((((((Real.0 <= two_pi) and Real.0 <= from_nat[Real](k.suc)) and continuous(g_cos_nt(k.suc))) and is_derivative_fn(g_cos_nt(k.suc), cos_nt(k.suc))) and forall(u: Real, v: Real) {

                interval_contains(Real.0, two_pi, u) and interval_contains(Real.0, two_pi, v)
                implies (cos_nt(k.suc, u) - cos_nt(k.suc, v)).abs <= from_nat[Real](k.suc) * (u - v).abs
            }) and forall(t: Real) { interval_contains(Real.0, two_pi, t) implies -Real.1 <= cos_nt(k.suc, t) }) and forall(t: Real) { interval_contains(Real.0, two_pi, t) implies cos_nt(k.suc, t) <= Real.1 }

        if ((((((Real.0 <= two_pi) and Real.0 <= from_nat[Real](k.suc)) and continuous(g_cos_nt(k.suc))) and is_derivative_fn(g_cos_nt(k.suc), cos_nt(k.suc))) and forall(u: Real, v: Real) {

                interval_contains(Real.0, two_pi, u) and interval_contains(Real.0, two_pi, v)
                implies (cos_nt(k.suc, u) - cos_nt(k.suc, v)).abs <= from_nat[Real](k.suc) * (u - v).abs
            }) and forall(t: Real) { interval_contains(Real.0, two_pi, t) implies -Real.1 <= cos_nt(k.suc, t) }) and forall(t: Real) { interval_contains(Real.0, two_pi, t) implies cos_nt(k.suc, t) <= Real.1 } {

            fn_integrable_gen(cos_nt(k.suc), g_cos_nt(k.suc), Real.0, two_pi, from_nat[Real](k.suc), -Real.1, Real.1)
            is_integrable(cos_nt(k.suc), Real.0, two_pi)
        }
}

/// The integral of kx.cos over [0, 2*pi] is zero for every k >= 1.
theorem integral_cos_nt_zero(k: Nat) {
    integral(cos_nt(k.suc), Real.0, two_pi) = Real.0
} by {
        two_pi_nonneg
        Real.0 <= two_pi
        g_cos_nt_continuous(k.suc)
        continuous(g_cos_nt(k.suc))
        g_cos_nt_has_derivative(k.suc)
        lt_add_suc(Nat.0, k)
        Nat.0 < Nat.0 + k.suc
        Nat.0 + k.suc = k.suc
        Nat.0 < k.suc
        Nat.0 < k.suc implies is_derivative_fn(g_cos_nt(k.suc), cos_nt(k.suc))
        is_derivative_fn(g_cos_nt(k.suc), cos_nt(k.suc))
        cos_nt_integrable(k)
        is_integrable(cos_nt(k.suc), Real.0, two_pi)
        forall(t: Real) {
            if interval_contains(Real.0, two_pi, t) {
                cos_nt_abs_le_one(k.suc, t)
                (cos_nt(k.suc, t)).abs <= Real.1
                abs_le_one_imp_lower(cos_nt(k.suc, t))
                -Real.1 <= cos_nt(k.suc, t)
            }
        }
        forall(t: Real) {
            if interval_contains(Real.0, two_pi, t) {
                cos_nt_abs_le_one(k.suc, t)
                (cos_nt(k.suc, t)).abs <= Real.1
                abs_le_one_imp_upper(cos_nt(k.suc, t))
                cos_nt(k.suc, t) <= Real.1
            }
        }
        Real.0 <= two_pi
        (Real.0 <= two_pi) and continuous(g_cos_nt(k.suc))
        ((Real.0 <= two_pi) and continuous(g_cos_nt(k.suc))) and is_derivative_fn(g_cos_nt(k.suc), cos_nt(k.suc))
        (((Real.0 <= two_pi) and continuous(g_cos_nt(k.suc))) and is_derivative_fn(g_cos_nt(k.suc), cos_nt(k.suc))) and is_integrable(cos_nt(k.suc), Real.0, two_pi)

        ((((Real.0 <= two_pi) and continuous(g_cos_nt(k.suc))) and is_derivative_fn(g_cos_nt(k.suc), cos_nt(k.suc))) and is_integrable(cos_nt(k.suc), Real.0, two_pi)) and forall(t: Real) {

                interval_contains(Real.0, two_pi, t) implies -Real.1 <= cos_nt(k.suc, t)
            }
        (((((Real.0 <= two_pi) and continuous(g_cos_nt(k.suc))) and is_derivative_fn(g_cos_nt(k.suc), cos_nt(k.suc))) and is_integrable(cos_nt(k.suc), Real.0, two_pi)) and forall(t: Real) {

                interval_contains(Real.0, two_pi, t) implies -Real.1 <= cos_nt(k.suc, t)
            }) and forall(t: Real) { interval_contains(Real.0, two_pi, t) implies cos_nt(k.suc, t) <= Real.1 }
        if (((((Real.0 <= two_pi) and continuous(g_cos_nt(k.suc))) and is_derivative_fn(g_cos_nt(k.suc), cos_nt(k.suc))) and is_integrable(cos_nt(k.suc), Real.0, two_pi)) and forall(t: Real) {

                interval_contains(Real.0, two_pi, t) implies -Real.1 <= cos_nt(k.suc, t)
            }) and forall(t: Real) { interval_contains(Real.0, two_pi, t) implies cos_nt(k.suc, t) <= Real.1 } {
            ftc2_general(cos_nt(k.suc), g_cos_nt(k.suc), Real.0, two_pi, -Real.1, Real.1)
            integral(cos_nt(k.suc), Real.0, two_pi) = g_cos_nt(k.suc)(two_pi) - g_cos_nt(k.suc)(Real.0)
            pointwise_mul_apply(constant[Real, Real](Real.1 / from_nat[Real](k.suc)), sin_nt(k.suc), two_pi)
            pointwise_mul(constant[Real, Real](Real.1 / from_nat[Real](k.suc)), sin_nt(k.suc), two_pi) = constant[Real, Real](Real.1 / from_nat[Real](k.suc), two_pi) * sin_nt(k.suc, two_pi)

            constant[Real, Real](Real.1 / from_nat[Real](k.suc), two_pi) = Real.1 / from_nat[Real](k.suc)
            pointwise_mul(constant[Real, Real](Real.1 / from_nat[Real](k.suc)), sin_nt(k.suc), two_pi) = (Real.1 / from_nat[Real](k.suc)) * sin_nt(k.suc, two_pi)

            g_cos_nt(k.suc)(two_pi) = (Real.1 / from_nat[Real](k.suc)) * sin_nt(k.suc, two_pi)
            sin_nt_two_pi_zero(k.suc)
            sin_nt(k.suc, two_pi) = Real.0
            (Real.1 / from_nat[Real](k.suc)) * sin_nt(k.suc, two_pi) = Real.0
            g_cos_nt(k.suc)(two_pi) = Real.0
            pointwise_mul_apply(constant[Real, Real](Real.1 / from_nat[Real](k.suc)), sin_nt(k.suc), Real.0)
            pointwise_mul(constant[Real, Real](Real.1 / from_nat[Real](k.suc)), sin_nt(k.suc), Real.0) = constant[Real, Real](Real.1 / from_nat[Real](k.suc), Real.0) * sin_nt(k.suc, Real.0)

            constant[Real, Real](Real.1 / from_nat[Real](k.suc), Real.0) = Real.1 / from_nat[Real](k.suc)
            pointwise_mul(constant[Real, Real](Real.1 / from_nat[Real](k.suc)), sin_nt(k.suc), Real.0) = (Real.1 / from_nat[Real](k.suc)) * sin_nt(k.suc, Real.0)

            g_cos_nt(k.suc)(Real.0) = (Real.1 / from_nat[Real](k.suc)) * sin_nt(k.suc, Real.0)
            sin_nt_at_zero(k.suc)
            sin_nt(k.suc, Real.0) = Real.0
            (Real.1 / from_nat[Real](k.suc)) * sin_nt(k.suc, Real.0) = Real.0
            g_cos_nt(k.suc)(Real.0) = Real.0
            g_cos_nt(k.suc)(two_pi) - g_cos_nt(k.suc)(Real.0) = Real.0 - Real.0
            Real.0 - Real.0 = Real.0
            integral(cos_nt(k.suc), Real.0, two_pi) = Real.0
        }
}

/// The integral of the constant one over [0, 2*pi] is 2*pi.
theorem integral_one_zero_two_pi {
    integral(constant[Real, Real](Real.1), Real.0, two_pi) = two_pi
} by {
    two_pi_nonneg
    Real.0 <= two_pi
    identity_function_is_continuous
    continuous(identity_fn[Real])
    derivative_fn_identity
    is_derivative_fn(identity_fn[Real], constant[Real, Real](Real.1))
    forall(u: Real, v: Real) {
        if interval_contains(Real.0, two_pi, u) and interval_contains(Real.0, two_pi, v) {
            constant[Real, Real](Real.1, u) = Real.1
            constant[Real, Real](Real.1, v) = Real.1
            constant[Real, Real](Real.1, u) - constant[Real, Real](Real.1, v) = Real.1 - Real.1
            Real.1 - Real.1 = Real.0
            (constant[Real, Real](Real.1, u) - constant[Real, Real](Real.1, v)).abs = Real.0.abs
            lte_self(Real.0)
            Real.0 <= Real.0
            abs_of_nonneg(Real.0)
            Real.0 >= Real.0 implies Real.0.abs = Real.0
            Real.0.abs = Real.0
            (constant[Real, Real](Real.1, u) - constant[Real, Real](Real.1, v)).abs = Real.0
            Real.0 * (u - v).abs = Real.0
            Real.0 <= Real.0
            Real.0 <= Real.0 * (u - v).abs
            (constant[Real, Real](Real.1, u) - constant[Real, Real](Real.1, v)).abs <= Real.0 * (u - v).abs

        }
    }
    forall(t: Real) {
        if interval_contains(Real.0, two_pi, t) {
            constant[Real, Real](Real.1, t) = Real.1
            Real.1 <= Real.1
            Real.1 <= constant[Real, Real](Real.1, t)
        }
    }
    forall(t: Real) {
        if interval_contains(Real.0, two_pi, t) {
            constant[Real, Real](Real.1, t) = Real.1
            Real.1 <= Real.1
            constant[Real, Real](Real.1, t) <= Real.1
        }
    }
    Real.0 <= two_pi
    (Real.0 <= two_pi) and Real.0 <= Real.0
    ((Real.0 <= two_pi) and Real.0 <= Real.0) and continuous(identity_fn[Real])
    (((Real.0 <= two_pi) and Real.0 <= Real.0) and continuous(identity_fn[Real])) and is_derivative_fn(identity_fn[Real], constant[Real, Real](Real.1))

    ((((Real.0 <= two_pi) and Real.0 <= Real.0) and continuous(identity_fn[Real])) and is_derivative_fn(identity_fn[Real], constant[Real, Real](Real.1))) and forall(u: Real, v: Real) {

            interval_contains(Real.0, two_pi, u) and interval_contains(Real.0, two_pi, v)
            implies (constant[Real, Real](Real.1, u) - constant[Real, Real](Real.1, v)).abs <= Real.0 * (u - v).abs

        }
    (((((Real.0 <= two_pi) and Real.0 <= Real.0) and continuous(identity_fn[Real])) and is_derivative_fn(identity_fn[Real], constant[Real, Real](Real.1))) and forall(u: Real, v: Real) {

            interval_contains(Real.0, two_pi, u) and interval_contains(Real.0, two_pi, v)
            implies (constant[Real, Real](Real.1, u) - constant[Real, Real](Real.1, v)).abs <= Real.0 * (u - v).abs

        }) and forall(t: Real) { interval_contains(Real.0, two_pi, t) implies Real.1 <= constant[Real, Real](Real.1, t) }
    ((((((Real.0 <= two_pi) and Real.0 <= Real.0) and continuous(identity_fn[Real])) and is_derivative_fn(identity_fn[Real], constant[Real, Real](Real.1))) and forall(u: Real, v: Real) {

            interval_contains(Real.0, two_pi, u) and interval_contains(Real.0, two_pi, v)
            implies (constant[Real, Real](Real.1, u) - constant[Real, Real](Real.1, v)).abs <= Real.0 * (u - v).abs

        }) and forall(t: Real) { interval_contains(Real.0, two_pi, t) implies Real.1 <= constant[Real, Real](Real.1, t) }) and forall(t: Real) { interval_contains(Real.0, two_pi, t) implies constant[Real, Real](Real.1, t) <= Real.1 }

    if ((((((Real.0 <= two_pi) and Real.0 <= Real.0) and continuous(identity_fn[Real])) and is_derivative_fn(identity_fn[Real], constant[Real, Real](Real.1))) and forall(u: Real, v: Real) {

            interval_contains(Real.0, two_pi, u) and interval_contains(Real.0, two_pi, v)
            implies (constant[Real, Real](Real.1, u) - constant[Real, Real](Real.1, v)).abs <= Real.0 * (u - v).abs

        }) and forall(t: Real) { interval_contains(Real.0, two_pi, t) implies Real.1 <= constant[Real, Real](Real.1, t) }) and forall(t: Real) { interval_contains(Real.0, two_pi, t) implies constant[Real, Real](Real.1, t) <= Real.1 } {

        fn_integrable_gen(constant[Real, Real](Real.1), identity_fn[Real], Real.0, two_pi, Real.0, Real.1, Real.1)

        is_integrable(constant[Real, Real](Real.1), Real.0, two_pi)
    }
    two_pi_nonneg
    Real.0 <= two_pi
    identity_function_is_continuous
    continuous(identity_fn[Real])
    derivative_fn_identity
    is_derivative_fn(identity_fn[Real], constant[Real, Real](Real.1))
    forall(t: Real) {
        if interval_contains(Real.0, two_pi, t) {
            constant[Real, Real](Real.1, t) = Real.1
            Real.1 <= Real.1
            Real.1 <= constant[Real, Real](Real.1, t)
        }
    }
    forall(t: Real) {
        if interval_contains(Real.0, two_pi, t) {
            constant[Real, Real](Real.1, t) = Real.1
            Real.1 <= Real.1
            constant[Real, Real](Real.1, t) <= Real.1
        }
    }
    Real.0 <= two_pi
    (Real.0 <= two_pi) and continuous(identity_fn[Real])
    ((Real.0 <= two_pi) and continuous(identity_fn[Real])) and is_derivative_fn(identity_fn[Real], constant[Real, Real](Real.1))
    (((Real.0 <= two_pi) and continuous(identity_fn[Real])) and is_derivative_fn(identity_fn[Real], constant[Real, Real](Real.1))) and is_integrable(constant[Real, Real](Real.1), Real.0, two_pi)

    ((((Real.0 <= two_pi) and continuous(identity_fn[Real])) and is_derivative_fn(identity_fn[Real], constant[Real, Real](Real.1))) and is_integrable(constant[Real, Real](Real.1), Real.0, two_pi)) and forall(t: Real) {

            interval_contains(Real.0, two_pi, t) implies Real.1 <= constant[Real, Real](Real.1, t)
        }
    (((((Real.0 <= two_pi) and continuous(identity_fn[Real])) and is_derivative_fn(identity_fn[Real], constant[Real, Real](Real.1))) and is_integrable(constant[Real, Real](Real.1), Real.0, two_pi)) and forall(t: Real) {

            interval_contains(Real.0, two_pi, t) implies Real.1 <= constant[Real, Real](Real.1, t)
        }) and forall(t: Real) { interval_contains(Real.0, two_pi, t) implies constant[Real, Real](Real.1, t) <= Real.1 }
    if (((((Real.0 <= two_pi) and continuous(identity_fn[Real])) and is_derivative_fn(identity_fn[Real], constant[Real, Real](Real.1))) and is_integrable(constant[Real, Real](Real.1), Real.0, two_pi)) and forall(t: Real) {

            interval_contains(Real.0, two_pi, t) implies Real.1 <= constant[Real, Real](Real.1, t)
        }) and forall(t: Real) { interval_contains(Real.0, two_pi, t) implies constant[Real, Real](Real.1, t) <= Real.1 } {
        ftc2_general(constant[Real, Real](Real.1), identity_fn[Real], Real.0, two_pi, Real.1, Real.1)
        integral(constant[Real, Real](Real.1), Real.0, two_pi) = identity_fn[Real](two_pi) - identity_fn[Real](Real.0)
        identity_fn[Real](two_pi) = two_pi
        identity_fn[Real](Real.0) = Real.0
        identity_fn[Real](two_pi) - identity_fn[Real](Real.0) = two_pi - Real.0
        two_pi - Real.0 = two_pi
        integral(constant[Real, Real](Real.1), Real.0, two_pi) = two_pi
    }
}

// ---------------------------------------------------------------------------
// The Dirichlet kernel D_n(t) = 1 + 2 sum_{k=1}^{n} kt.cos
// ---------------------------------------------------------------------------

/// The real Dirichlet kernel of order n: D_n(t) = 1 + 2 sum_{k=1}^{n} kt.cos.
define dirichlet_kernel(t: Real, n: Nat) -> Real {
    Real.1 + two * cos_harmonic_sum(t, n)
}

/// (t/2).sin * D_n(t) = ((n + 1/2)t).sin.
theorem dirichlet_kernel_scaled(t: Real, n: Nat) {
    (t * Real.one_half).sin * dirichlet_kernel(t, n) = half_angle_harmonic(t, n)
} by {
    dirichlet_kernel(t, n) = Real.1 + two * cos_harmonic_sum(t, n)
    mul_distrib_left(two, Real.one_half, cos_harmonic_sum(t, n))
    two * (Real.one_half + cos_harmonic_sum(t, n)) = two * Real.one_half + two * cos_harmonic_sum(t, n)

    one_half_plus_one_half
    Real.one_half + Real.one_half = Real.1
    two * Real.one_half = Real.1
    two * Real.one_half + two * cos_harmonic_sum(t, n) = Real.1 + two * cos_harmonic_sum(t, n)
    two * (Real.one_half + cos_harmonic_sum(t, n)) = Real.1 + two * cos_harmonic_sum(t, n)
    Real.1 + two * cos_harmonic_sum(t, n) = two * (Real.one_half + cos_harmonic_sum(t, n))
    dirichlet_kernel(t, n) = two * (Real.one_half + cos_harmonic_sum(t, n))
    (t * Real.one_half).sin * dirichlet_kernel(t, n) = (t * Real.one_half).sin * (two * (Real.one_half + cos_harmonic_sum(t, n)))

    mul_assoc((t * Real.one_half).sin, two, Real.one_half + cos_harmonic_sum(t, n))
    (t * Real.one_half).sin * (two * (Real.one_half + cos_harmonic_sum(t, n))) = ((t * Real.one_half).sin * two) * (Real.one_half + cos_harmonic_sum(t, n))

    real_mul_comm((t * Real.one_half).sin, two)
    (t * Real.one_half).sin * two = two * (t * Real.one_half).sin
    ((t * Real.one_half).sin * two) * (Real.one_half + cos_harmonic_sum(t, n)) = (two * (t * Real.one_half).sin) * (Real.one_half + cos_harmonic_sum(t, n))

    (t * Real.one_half).sin * (two * (Real.one_half + cos_harmonic_sum(t, n))) = (two * (t * Real.one_half).sin) * (Real.one_half + cos_harmonic_sum(t, n))

    (t * Real.one_half).sin * dirichlet_kernel(t, n) = (two * (t * Real.one_half).sin) * (Real.one_half + cos_harmonic_sum(t, n))

    dirichlet_kernel_identity(t, n)
    two * (t * Real.one_half).sin * (Real.one_half + cos_harmonic_sum(t, n)) = half_angle_harmonic(t, n)
    (two * (t * Real.one_half).sin) * (Real.one_half + cos_harmonic_sum(t, n)) = half_angle_harmonic(t, n)

    (t * Real.one_half).sin * dirichlet_kernel(t, n) = half_angle_harmonic(t, n)
}

/// The closed form of the Dirichlet kernel away from the zeros of (t/2).sin:
/// D_n(t) = ((n + 1/2)t).sin / (t/2).sin.
theorem dirichlet_kernel_closed_form(t: Real, n: Nat) {
    (t * Real.one_half).sin != Real.0
    implies dirichlet_kernel(t, n) = half_angle_harmonic(t, n) / (t * Real.one_half).sin
} by {
    if (t * Real.one_half).sin != Real.0 {
        dirichlet_kernel_scaled(t, n)
        (t * Real.one_half).sin * dirichlet_kernel(t, n) = half_angle_harmonic(t, n)
        half_angle_harmonic(t, n) / (t * Real.one_half).sin = ((t * Real.one_half).sin * dirichlet_kernel(t, n)) / (t * Real.one_half).sin

        div_mul_cancel_left((t * Real.one_half).sin, dirichlet_kernel(t, n))
        (t * Real.one_half).sin != Real.0 implies ((t * Real.one_half).sin * dirichlet_kernel(t, n)) / (t * Real.one_half).sin =  dirichlet_kernel(t, n)

        ((t * Real.one_half).sin * dirichlet_kernel(t, n)) / (t * Real.one_half).sin = dirichlet_kernel(t, n)

        half_angle_harmonic(t, n) / (t * Real.one_half).sin = dirichlet_kernel(t, n)
        dirichlet_kernel(t, n) = half_angle_harmonic(t, n) / (t * Real.one_half).sin
    }
}

// ---------------------------------------------------------------------------
// Partial sums of sequences of functions
// ---------------------------------------------------------------------------

/// The pointwise application of a sequence of functions at x.
define seq_apply(seq: Nat -> (Real -> Real), x: Real, i: Nat) -> Real {
    seq(i, x)
}

/// The partial sum of the first n terms of a sequence of functions, as a
/// function: sum_{i < n} seq(i)(x).
define fn_partial_seq(seq: Nat -> (Real -> Real), n: Nat, x: Real) -> Real {
    partial(seq_apply(seq, x), n)
}

/// The pointwise absolute value of a real sequence.
define seq_abs(a: Nat -> Real, i: Nat) -> Real {
    (a(i)).abs
}

/// The triangle inequality for partial sums: |sum a_i| <= sum |a_i|.
theorem partial_triangle_abs(a: Nat -> Real, n: Nat) {
    partial(a, n).abs <= partial(seq_abs(a), n)
} by {
    define q(m: Nat) -> Bool {
        partial(a, m).abs <= partial(seq_abs(a), m)
    }
    partial_zero(a)
    partial(a, Nat.0) = Real.0
    partial_zero(seq_abs(a))
    partial(seq_abs(a), Nat.0) = Real.0
    partial(a, Nat.0).abs = Real.0.abs
    lte_self(Real.0)
    Real.0 <= Real.0
    abs_of_nonneg(Real.0)
    Real.0 >= Real.0 implies Real.0.abs = Real.0
    Real.0.abs = Real.0
    partial(a, Nat.0).abs = Real.0
    Real.0 <= partial(seq_abs(a), Nat.0)
    partial(a, Nat.0).abs <= partial(seq_abs(a), Nat.0)
    q(Nat.0)
    forall(m: Nat) {
        if q(m) {
            partial_suc(a, m)
            partial(a, m.suc) = partial(a, m) + a(m)
            partial_suc(seq_abs(a), m)
            partial(seq_abs(a), m.suc) = partial(seq_abs(a), m) + seq_abs(a, m)
            seq_abs(a, m) = (a(m)).abs
            partial(seq_abs(a), m.suc) = partial(seq_abs(a), m) + (a(m)).abs
            triangle_ineq(partial(a, m), a(m))
            (partial(a, m) + a(m)).abs <= partial(a, m).abs + (a(m)).abs
            partial(a, m.suc).abs <= partial(a, m).abs + (a(m)).abs
            partial(a, m).abs <= partial(seq_abs(a), m)
            add_le_add(partial(a, m).abs, partial(seq_abs(a), m), (a(m)).abs, (a(m)).abs)
            partial(a, m).abs + (a(m)).abs <= partial(seq_abs(a), m) + (a(m)).abs
            lte_trans[Real](partial(a, m.suc).abs, partial(a, m).abs + (a(m)).abs, partial(seq_abs(a), m) + (a(m)).abs)

            partial(a, m.suc).abs <= partial(seq_abs(a), m) + (a(m)).abs
            partial(a, m.suc).abs <= partial(seq_abs(a), m.suc)
            q(m.suc)
        }
    }
    alt_induction(q)
    (q(Nat.0) and forall(j: Nat) { q(j) implies q(j.suc) }) implies forall(j: Nat) { q(j) }
    q(Nat.0) and forall(j: Nat) { q(j) implies q(j.suc) }
    forall(j: Nat) { q(j) }
    q(n)
}

/// The derivative of the partial sum of a sequence of functions is the
/// partial sum of the derivatives, for sequences of functions with known
/// pointwise derivatives.
theorem derivative_fn_fn_partial_seq(seq: Nat -> (Real -> Real), dseq: Nat -> (Real -> Real), n: Nat) {
    (forall(i: Nat) { i < n implies is_derivative_fn(seq(i), dseq(i)) })
    implies is_derivative_fn(fn_partial_seq(seq, n), fn_partial_seq(dseq, n))
} by {
    define q(m: Nat) -> Bool {
        (forall(i: Nat) { i < m implies is_derivative_fn(seq(i), dseq(i)) })
        implies is_derivative_fn(fn_partial_seq(seq, m), fn_partial_seq(dseq, m))
    }
    // base case: the empty partial sum is constant zero.
    forall(x: Real) {
        partial_zero(seq_apply(seq, x))
        partial(seq_apply(seq, x), Nat.0) = Real.0
        fn_partial_seq(seq, Nat.0, x) = partial(seq_apply(seq, x), Nat.0)
        fn_partial_seq(seq, Nat.0, x) = Real.0
        constant[Real, Real](Real.0, x) = Real.0
        fn_partial_seq(seq, Nat.0, x) = constant[Real, Real](Real.0, x)
    }
    function_extensionality(fn_partial_seq(seq, Nat.0), constant[Real, Real](Real.0))
    forall(x: Real) {
        partial_zero(seq_apply(dseq, x))
        partial(seq_apply(dseq, x), Nat.0) = Real.0
        fn_partial_seq(dseq, Nat.0, x) = partial(seq_apply(dseq, x), Nat.0)
        fn_partial_seq(dseq, Nat.0, x) = Real.0
        constant[Real, Real](Real.0, x) = Real.0
        fn_partial_seq(dseq, Nat.0, x) = constant[Real, Real](Real.0, x)
    }
    fn_partial_seq(seq, Nat.0) = constant[Real, Real](Real.0)
    function_extensionality(fn_partial_seq(dseq, Nat.0), constant[Real, Real](Real.0))
    fn_partial_seq(dseq, Nat.0) = constant[Real, Real](Real.0)
    derivative_fn_constant(Real.0)
    is_derivative_fn(constant[Real, Real](Real.0), constant[Real, Real](Real.0))
    is_derivative_fn(fn_partial_seq(seq, Nat.0), fn_partial_seq(dseq, Nat.0))
    q(Nat.0)
    // inductive step
    forall(m: Nat) {
        if q(m) {
            if forall(i: Nat) { i < m.suc implies is_derivative_fn(seq(i), dseq(i)) } {
                lt_suc(m)
                m < m.suc
                forall(i: Nat) {
                    if i < m {
                        lt_trans(i, m, m.suc)
                        i < m and m < m.suc implies i < m.suc
                        i < m.suc
                        is_derivative_fn(seq(i), dseq(i))
                    }
                }
                forall(i: Nat) { i < m implies is_derivative_fn(seq(i), dseq(i)) }
                q(m) = ((forall(i: Nat) { i < m implies is_derivative_fn(seq(i), dseq(i)) }) implies is_derivative_fn(fn_partial_seq(seq, m), fn_partial_seq(dseq, m)))
                (forall(i: Nat) { i < m implies is_derivative_fn(seq(i), dseq(i)) }) implies is_derivative_fn(fn_partial_seq(seq, m), fn_partial_seq(dseq, m))
                is_derivative_fn(fn_partial_seq(seq, m), fn_partial_seq(dseq, m))
                is_derivative_fn(seq(m), dseq(m))
                derivative_fn_add(fn_partial_seq(seq, m), seq(m), fn_partial_seq(dseq, m), dseq(m))
                is_derivative_fn(pointwise_add(fn_partial_seq(seq, m), seq(m)), pointwise_add(fn_partial_seq(dseq, m), dseq(m)))

                forall(x: Real) {
                    partial_suc(seq_apply(seq, x), m)
                    partial(seq_apply(seq, x), m.suc) = partial(seq_apply(seq, x), m) + seq_apply(seq, x, m)
                    fn_partial_seq(seq, m.suc, x) = partial(seq_apply(seq, x), m.suc)
                    fn_partial_seq(seq, m.suc, x) = partial(seq_apply(seq, x), m) + seq_apply(seq, x, m)
                    fn_partial_seq(seq, m, x) = partial(seq_apply(seq, x), m)
                    fn_partial_seq(seq, m.suc, x) = fn_partial_seq(seq, m, x) + seq_apply(seq, x, m)
                    seq_apply(seq, x, m) = seq(m, x)
                    fn_partial_seq(seq, m.suc, x) = fn_partial_seq(seq, m, x) + seq(m, x)
                    pointwise_add_apply(fn_partial_seq(seq, m), seq(m), x)
                    pointwise_add(fn_partial_seq(seq, m), seq(m), x) = fn_partial_seq(seq, m, x) + seq(m, x)

                    fn_partial_seq(seq, m.suc, x) = pointwise_add(fn_partial_seq(seq, m), seq(m), x)
                }
                function_extensionality(fn_partial_seq(seq, m.suc), pointwise_add(fn_partial_seq(seq, m), seq(m)))

                fn_partial_seq(seq, m.suc) = pointwise_add(fn_partial_seq(seq, m), seq(m))
                forall(x: Real) {
                    partial_suc(seq_apply(dseq, x), m)
                    partial(seq_apply(dseq, x), m.suc) = partial(seq_apply(dseq, x), m) + seq_apply(dseq, x, m)
                    fn_partial_seq(dseq, m.suc, x) = partial(seq_apply(dseq, x), m.suc)
                    fn_partial_seq(dseq, m.suc, x) = partial(seq_apply(dseq, x), m) + seq_apply(dseq, x, m)
                    fn_partial_seq(dseq, m, x) = partial(seq_apply(dseq, x), m)
                    fn_partial_seq(dseq, m.suc, x) = fn_partial_seq(dseq, m, x) + seq_apply(dseq, x, m)
                    seq_apply(dseq, x, m) = dseq(m, x)
                    fn_partial_seq(dseq, m.suc, x) = fn_partial_seq(dseq, m, x) + dseq(m, x)
                    pointwise_add_apply(fn_partial_seq(dseq, m), dseq(m), x)
                    pointwise_add(fn_partial_seq(dseq, m), dseq(m), x) = fn_partial_seq(dseq, m, x) + dseq(m, x)

                    fn_partial_seq(dseq, m.suc, x) = pointwise_add(fn_partial_seq(dseq, m), dseq(m), x)
                }
                function_extensionality(fn_partial_seq(dseq, m.suc), pointwise_add(fn_partial_seq(dseq, m), dseq(m)))

                fn_partial_seq(dseq, m.suc) = pointwise_add(fn_partial_seq(dseq, m), dseq(m))
                is_derivative_fn(fn_partial_seq(seq, m.suc), fn_partial_seq(dseq, m.suc))
                q(m.suc)
            }
        }
    }
    alt_induction(q)
    (q(Nat.0) and forall(j: Nat) { q(j) implies q(j.suc) }) implies forall(j: Nat) { q(j) }
    q(Nat.0) and forall(j: Nat) { q(j) implies q(j.suc) }
    forall(j: Nat) { q(j) }
    q(n)
}

// ---------------------------------------------------------------------------
// The antiderivative of the Dirichlet kernel
// ---------------------------------------------------------------------------

/// The sequence of cosine harmonics ((i+1)x).cos.
define seq_cos(i: Nat) -> (Real -> Real) {
    cos_nt(i.suc)
}

/// The sequence of antiderivatives ((i+1)x).sin/(i+1) of the cosine
/// harmonics.
define seq_sin_over_k(i: Nat) -> (Real -> Real) {
    pointwise_mul(constant[Real, Real](Real.1 / from_nat[Real](i.suc)), sin_nt(i.suc))
}

/// The derivative of ((i+1)x).sin/(i+1) is ((i+1)x).cos.
theorem seq_sin_over_k_derivative(i: Nat) {
    is_derivative_fn(seq_sin_over_k(i), cos_nt(i.suc))
} by {
    g_cos_nt_has_derivative(i.suc)
    Nat.0 < i.suc implies is_derivative_fn(g_cos_nt(i.suc), cos_nt(i.suc))
    lt_add_suc(Nat.0, i)
    Nat.0 < Nat.0 + i.suc
    Nat.0 + i.suc = i.suc
    Nat.0 < i.suc
    is_derivative_fn(g_cos_nt(i.suc), cos_nt(i.suc))
    g_cos_nt(i.suc) = pointwise_mul(constant[Real, Real](Real.1 / from_nat[Real](i.suc)), sin_nt(i.suc))
    is_derivative_fn(pointwise_mul(constant[Real, Real](Real.1 / from_nat[Real](i.suc)), sin_nt(i.suc)), cos_nt(i.suc))

    seq_sin_over_k(i) = pointwise_mul(constant[Real, Real](Real.1 / from_nat[Real](i.suc)), sin_nt(i.suc))
    is_derivative_fn(seq_sin_over_k(i), cos_nt(i.suc))
}

/// The pointwise sum of the first n cosine harmonics, as a function.
define dirichlet_deriv_fn(n: Nat) -> (Real -> Real) {
    pointwise_add(constant[Real, Real](Real.1), pointwise_mul(constant[Real, Real](two), fn_partial_seq(seq_cos, n)))

}

/// The Dirichlet kernel as a function of the variable.
define dirichlet_kernel_fn(n: Nat, t: Real) -> Real {
    dirichlet_kernel(t, n)
}

/// The sum of the first n cosine harmonics equals cos_harmonic_sum pointwise.
theorem fn_partial_seq_cos_eq_sum(t: Real, n: Nat) {
    fn_partial_seq(seq_cos, n, t) = cos_harmonic_sum(t, n)
} by {
    fn_partial_seq(seq_cos, n, t) = partial(seq_apply(seq_cos, t), n)
    forall(i: Nat) {
        if i < n {
            seq_apply(seq_cos, t, i) = seq_cos(i, t)
            seq_cos(i, t) = cos_nt(i.suc, t)
            seq_apply(seq_cos, t, i) = cos_nt(i.suc, t)
            cos_nt(i.suc, t) = (from_nat[Real](i.suc) * t).cos
            cos_harmonic(t, i.suc) = (from_nat[Real](i.suc) * t).cos
            cos_nt(i.suc, t) = cos_harmonic(t, i.suc)
            seq_apply(seq_cos, t, i) = cos_harmonic(t, i.suc)
            compose(cos_harmonic(t), Nat.suc)(i) = cos_harmonic(t, i.suc)
            seq_apply(seq_cos, t, i) = compose(cos_harmonic(t), Nat.suc)(i)
        }
    }
    partial_pointwise_eq(seq_apply(seq_cos, t), compose(cos_harmonic(t), Nat.suc), n)
    partial(seq_apply(seq_cos, t), n) = partial(compose(cos_harmonic(t), Nat.suc), n)
    cos_harmonic_sum(t, n) = partial(compose(cos_harmonic(t), Nat.suc), n)
    partial(compose(cos_harmonic(t), Nat.suc), n) = cos_harmonic_sum(t, n)
    partial(seq_apply(seq_cos, t), n) = cos_harmonic_sum(t, n)
    fn_partial_seq(seq_cos, n, t) = cos_harmonic_sum(t, n)
}

/// The derivative expression of the kernel equals the kernel function.
theorem dirichlet_deriv_fn_eq_kernel(n: Nat) {
    dirichlet_deriv_fn(n) = dirichlet_kernel_fn(n)
} by {
    forall(t: Real) {
        pointwise_add_apply(constant[Real, Real](Real.1), pointwise_mul(constant[Real, Real](two), fn_partial_seq(seq_cos, n)), t)

        pointwise_add(constant[Real, Real](Real.1), pointwise_mul(constant[Real, Real](two), fn_partial_seq(seq_cos, n)), t) = constant[Real, Real](Real.1, t) +


                pointwise_mul(constant[Real, Real](two), fn_partial_seq(seq_cos, n), t)
        constant[Real, Real](Real.1, t) = Real.1
        pointwise_mul_apply(constant[Real, Real](two), fn_partial_seq(seq_cos, n), t)
        pointwise_mul(constant[Real, Real](two), fn_partial_seq(seq_cos, n), t) = constant[Real, Real](two, t) * fn_partial_seq(seq_cos, n, t)

        constant[Real, Real](two, t) = two
        pointwise_mul(constant[Real, Real](two), fn_partial_seq(seq_cos, n), t) = two * fn_partial_seq(seq_cos, n, t)

        pointwise_add(constant[Real, Real](Real.1), pointwise_mul(constant[Real, Real](two), fn_partial_seq(seq_cos, n)), t) = Real.1 + two * fn_partial_seq(seq_cos, n, t)


        dirichlet_deriv_fn(n, t) = Real.1 + two * fn_partial_seq(seq_cos, n, t)
        fn_partial_seq_cos_eq_sum(t, n)
        fn_partial_seq(seq_cos, n, t) = cos_harmonic_sum(t, n)
        dirichlet_deriv_fn(n, t) = Real.1 + two * cos_harmonic_sum(t, n)
        dirichlet_kernel(t, n) = Real.1 + two * cos_harmonic_sum(t, n)
        Real.1 + two * cos_harmonic_sum(t, n) = dirichlet_kernel(t, n)
        dirichlet_deriv_fn(n, t) = dirichlet_kernel(t, n)
        dirichlet_kernel_fn(n, t) = dirichlet_kernel(t, n)
        dirichlet_deriv_fn(n, t) = dirichlet_kernel_fn(n, t)
    }
    function_extensionality(dirichlet_deriv_fn(n), dirichlet_kernel_fn(n))
    dirichlet_deriv_fn(n) = dirichlet_kernel_fn(n)
}

/// The antiderivative of the Dirichlet kernel of order n:
/// x -> x + 2 sum_{k=1}^{n} kx.sin/k.
define g_dirichlet(n: Nat) -> (Real -> Real) {
    pointwise_add(identity_fn[Real], pointwise_mul(constant[Real, Real](two), fn_partial_seq(seq_sin_over_k, n)))

}

/// The derivative of the Dirichlet-kernel antiderivative is the kernel.
theorem g_dirichlet_has_derivative(n: Nat) {
    is_derivative_fn(g_dirichlet(n), dirichlet_kernel_fn(n))
} by {
    forall(i: Nat) {
        if i < n {
            seq_sin_over_k_derivative(i)
            is_derivative_fn(seq_sin_over_k(i), cos_nt(i.suc))
            seq_cos(i) = cos_nt(i.suc)
            is_derivative_fn(seq_sin_over_k(i), seq_cos(i))
        }
    }
    derivative_fn_fn_partial_seq(seq_sin_over_k, seq_cos, n)
    forall(i: Nat) { i < n implies is_derivative_fn(seq_sin_over_k(i), seq_cos(i)) }
    is_derivative_fn(fn_partial_seq(seq_sin_over_k, n), fn_partial_seq(seq_cos, n))
    derivative_fn_const_mul(two, fn_partial_seq(seq_sin_over_k, n), fn_partial_seq(seq_cos, n))
    is_derivative_fn(pointwise_mul(constant[Real, Real](two), fn_partial_seq(seq_sin_over_k, n)), pointwise_mul(constant[Real, Real](two), fn_partial_seq(seq_cos, n)))

    derivative_fn_add(identity_fn[Real], pointwise_mul(constant[Real, Real](two), fn_partial_seq(seq_sin_over_k, n)), constant[Real, Real](Real.1), pointwise_mul(constant[Real, Real](two), fn_partial_seq(seq_cos, n)))



    is_derivative_fn(pointwise_add(identity_fn[Real], pointwise_mul(constant[Real, Real](two), fn_partial_seq(seq_sin_over_k, n))), dirichlet_deriv_fn(n))


    g_dirichlet(n) = pointwise_add(identity_fn[Real], pointwise_mul(constant[Real, Real](two), fn_partial_seq(seq_sin_over_k, n)))

    is_derivative_fn(g_dirichlet(n), dirichlet_deriv_fn(n))
    dirichlet_deriv_fn_eq_kernel(n)
    dirichlet_deriv_fn(n) = dirichlet_kernel_fn(n)
    is_derivative_fn(g_dirichlet(n), dirichlet_kernel_fn(n))
}

// ---------------------------------------------------------------------------
// Partial-sum comparison lemmas
// ---------------------------------------------------------------------------

/// The partial sum of a constant-valued sequence is the count times the value.
theorem partial_const_seq(f: Nat -> Real, n: Nat, c: Real) {
    (forall(i: Nat) { i < n implies f(i) = c }) implies partial(f, n) = from_nat[Real](n) * c
} by {
    define q(m: Nat) -> Bool {
        (forall(i: Nat) { i < m implies f(i) = c }) implies partial(f, m) = from_nat[Real](m) * c
    }
    partial_zero(f)
    partial(f, Nat.0) = Real.0
    from_nat_zero[Real]
    from_nat[Real](Nat.0) = Real.0
    from_nat[Real](Nat.0) * c = Real.0 * c
    Real.0 * c = Real.0
    from_nat[Real](Nat.0) * c = Real.0
    partial(f, Nat.0) = from_nat[Real](Nat.0) * c
    q(Nat.0)
    forall(m: Nat) {
        if q(m) {
            if forall(i: Nat) { i < m.suc implies f(i) = c } {
                lt_suc(m)
                m < m.suc
                forall(i: Nat) {
                    if i < m {
                        lt_trans(i, m, m.suc)
                        i < m and m < m.suc implies i < m.suc
                        i < m.suc
                        f(i) = c
                    }
                }
                forall(i: Nat) { i < m implies f(i) = c }
                q(m) = ((forall(i: Nat) { i < m implies f(i) = c }) implies partial(f, m) = from_nat[Real](m) * c)
                (forall(i: Nat) { i < m implies f(i) = c }) implies partial(f, m) = from_nat[Real](m) * c
                partial(f, m) = from_nat[Real](m) * c
                partial_suc(f, m)
                partial(f, m.suc) = partial(f, m) + f(m)
                f(m) = c
                partial(f, m.suc) = from_nat[Real](m) * c + c
                from_nat_add[Real](m, Nat.1)
                from_nat[Real](m + Nat.1) = from_nat[Real](m) + from_nat[Real](Nat.1)
                m + Nat.1 = m.suc
                from_nat[Real](m.suc) = from_nat[Real](m) + from_nat[Real](Nat.1)
                from_nat[Real](Nat.1) = Real.1
                from_nat[Real](m.suc) = from_nat[Real](m) + Real.1
                mul_distrib_left(from_nat[Real](m), Real.1, c)
                (from_nat[Real](m) + Real.1) * c = from_nat[Real](m) * c + Real.1 * c
                Real.1 * c = c
                (from_nat[Real](m) + Real.1) * c = from_nat[Real](m) * c + c
                from_nat[Real](m.suc) * c = from_nat[Real](m) * c + c
                partial(f, m.suc) = from_nat[Real](m.suc) * c
                q(m.suc)
            }
        }
    }
    alt_induction(q)
    (q(Nat.0) and forall(j: Nat) { q(j) implies q(j.suc) }) implies forall(j: Nat) { q(j) }
    q(Nat.0) and forall(j: Nat) { q(j) implies q(j.suc) }
    forall(j: Nat) { q(j) }
    q(n)
}

/// Partial sums are monotone pointwise.
theorem partial_lte_self(f: Nat -> Real, g: Nat -> Real, n: Nat) {
    (forall(i: Nat) { i < n implies f(i) <= g(i) }) implies partial(f, n) <= partial(g, n)
} by {
    define q(m: Nat) -> Bool {
        (forall(i: Nat) { i < m implies f(i) <= g(i) }) implies partial(f, m) <= partial(g, m)
    }
    partial_zero(f)
    partial(f, Nat.0) = Real.0
    partial_zero(g)
    partial(g, Nat.0) = Real.0
    lte_self(Real.0)
    Real.0 <= Real.0
    partial(f, Nat.0) <= partial(g, Nat.0)
    q(Nat.0)
    forall(m: Nat) {
        if q(m) {
            if forall(i: Nat) { i < m.suc implies f(i) <= g(i) } {
                lt_suc(m)
                m < m.suc
                forall(i: Nat) {
                    if i < m {
                        lt_trans(i, m, m.suc)
                        i < m and m < m.suc implies i < m.suc
                        i < m.suc
                        f(i) <= g(i)
                    }
                }
                forall(i: Nat) { i < m implies f(i) <= g(i) }
                q(m) = ((forall(i: Nat) { i < m implies f(i) <= g(i) }) implies partial(f, m) <= partial(g, m))
                (forall(i: Nat) { i < m implies f(i) <= g(i) }) implies partial(f, m) <= partial(g, m)
                partial(f, m) <= partial(g, m)
                f(m) <= g(m)
                add_le_add(partial(f, m), partial(g, m), f(m), g(m))
                partial(f, m) + f(m) <= partial(g, m) + g(m)
                partial_suc(f, m)
                partial(f, m.suc) = partial(f, m) + f(m)
                partial_suc(g, m)
                partial(g, m.suc) = partial(g, m) + g(m)
                partial(f, m.suc) <= partial(g, m.suc)
                q(m.suc)
            }
        }
    }
    alt_induction(q)
    (q(Nat.0) and forall(j: Nat) { q(j) implies q(j.suc) }) implies forall(j: Nat) { q(j) }
    q(Nat.0) and forall(j: Nat) { q(j) implies q(j.suc) }
    forall(j: Nat) { q(j) }
    q(n)
}

/// The sequence of negated terms.
define seq_neg(a: Nat -> Real, i: Nat) -> Real {
    -(a(i))
}

/// The pointwise difference of two sequences.
define seq_diff(a: Nat -> Real, b: Nat -> Real, i: Nat) -> Real {
    a(i) - b(i)
}

/// The sequence constantly one.
define seq_one(i: Nat) -> Real {
    Real.1
}

/// The sequence of positive integers k = i + 1.
define seq_k(i: Nat) -> Real {
    from_nat[Real](i.suc)
}

/// The partial sum of the negated sequence is the negation of the partial sum.
theorem partial_neg_seq(a: Nat -> Real, n: Nat) {
    partial(seq_neg(a), n) = -(partial(a, n))
} by {
    partial_scalar_mul(-Real.1, a, n)
    (-Real.1) * partial(a, n) = partial(mul_fn(-Real.1, a), n)
    forall(i: Nat) {
        if i < n {
            mul_fn(-Real.1, a, i) = (-Real.1) * a(i)
            real_mul_comm(-Real.1, a(i))
            (-Real.1) * a(i) = a(i) * (-Real.1)
            mul_neg_one_right(a(i))
            a(i) * (-Real.1) = -(a(i))
            (-Real.1) * a(i) = -(a(i))
            mul_fn(-Real.1, a, i) = -(a(i))
            seq_neg(a, i) = -(a(i))
            mul_fn(-Real.1, a, i) = seq_neg(a, i)
        }
    }
    partial_pointwise_eq(mul_fn(-Real.1, a), seq_neg(a), n)
    partial(mul_fn(-Real.1, a), n) = partial(seq_neg(a), n)
    (-Real.1) * partial(a, n) = partial(seq_neg(a), n)
    mul_neg_one_left(partial(a, n))
    (-Real.1) * partial(a, n) = -(partial(a, n))
    partial(seq_neg(a), n) = -(partial(a, n))
}

/// The partial sum of a difference is the difference of the partial sums.
theorem partial_diff_seq(a: Nat -> Real, b: Nat -> Real, n: Nat) {
    partial(a, n) - partial(b, n) = partial(seq_diff(a, b), n)
} by {
    partial_add(a, seq_neg(b), n)
    partial(a, n) + partial(seq_neg(b), n) = partial(add_fn(a, seq_neg(b)), n)
    partial_neg_seq(b, n)
    partial(seq_neg(b), n) = -(partial(b, n))
    partial(a, n) + (-(partial(b, n))) = partial(add_fn(a, seq_neg(b)), n)
    partial(a, n) - partial(b, n) = partial(add_fn(a, seq_neg(b)), n)
    forall(i: Nat) {
        if i < n {
            add_fn(a, seq_neg(b), i) = a(i) + seq_neg(b, i)
            seq_neg(b, i) = -(b(i))
            add_fn(a, seq_neg(b), i) = a(i) + (-(b(i)))
            add_fn(a, seq_neg(b), i) = a(i) - b(i)
            seq_diff(a, b, i) = a(i) - b(i)
            add_fn(a, seq_neg(b), i) = seq_diff(a, b, i)
        }
    }
    partial_pointwise_eq(add_fn(a, seq_neg(b)), seq_diff(a, b), n)
    partial(add_fn(a, seq_neg(b)), n) = partial(seq_diff(a, b), n)
    partial(a, n) - partial(b, n) = partial(seq_diff(a, b), n)
}

/// The partial sum of the sequence constantly one is from_nat(n).
theorem partial_seq_one(n: Nat) {
    partial(seq_one, n) = from_nat[Real](n)
} by {
    forall(i: Nat) {
        if i < n {
            seq_one(i) = Real.1
        }
    }
    partial_const_seq(seq_one, n, Real.1)
    forall(i: Nat) { i < n implies seq_one(i) = Real.1 }
    partial(seq_one, n) = from_nat[Real](n) * Real.1
    from_nat[Real](n) * Real.1 = from_nat[Real](n)
    partial(seq_one, n) = from_nat[Real](n)
}

/// The partial sum of the zero sequence is zero.
theorem partial_seq_zero(a: Nat -> Real, n: Nat) {
    (forall(i: Nat) { i < n implies a(i) = Real.0 }) implies partial(a, n) = Real.0
} by {
    if forall(i: Nat) { i < n implies a(i) = Real.0 } {
        partial_const_seq(a, n, Real.0)
        forall(i: Nat) { i < n implies a(i) = Real.0 }
        partial(a, n) = from_nat[Real](n) * Real.0
        from_nat[Real](n) * Real.0 = Real.0
        partial(a, n) = Real.0
    }
}

// ---------------------------------------------------------------------------
// The integral of the Dirichlet kernel over one period
// ---------------------------------------------------------------------------

/// The pointwise difference sequence of the cosine harmonics at u and v.
define seq_diff_cos(u: Real, v: Real, i: Nat) -> Real {
    cos_nt(i.suc, u) - cos_nt(i.suc, v)
}

/// The scaled difference sequence k * |u - v|.
define seq_k_absdiff(u: Real, v: Real, i: Nat) -> Real {
    from_nat[Real](i.suc) * (u - v).abs
}

/// The Dirichlet kernel is (2 sum_{k=1}^{n} k)-Lipschitz.
theorem dirichlet_kernel_fn_lipschitz(n: Nat, u: Real, v: Real) {
    (dirichlet_kernel_fn(n, u) - dirichlet_kernel_fn(n, v)).abs <= (two * partial(seq_k, n)) * (u - v).abs

} by {
    dirichlet_kernel_fn(n, u) = dirichlet_kernel(u, n)
    dirichlet_kernel(u, n) = Real.1 + two * cos_harmonic_sum(u, n)
    dirichlet_kernel_fn(n, u) = Real.1 + two * cos_harmonic_sum(u, n)
    dirichlet_kernel_fn(n, v) = dirichlet_kernel(v, n)
    dirichlet_kernel(v, n) = Real.1 + two * cos_harmonic_sum(v, n)
    dirichlet_kernel_fn(n, v) = Real.1 + two * cos_harmonic_sum(v, n)
    dirichlet_kernel_fn(n, u) - dirichlet_kernel_fn(n, v) = (Real.1 + two * cos_harmonic_sum(u, n)) - (Real.1 + two * cos_harmonic_sum(v, n))

    add_sub_add_cancel(Real.1, two * cos_harmonic_sum(u, n), two * cos_harmonic_sum(v, n))
    (Real.1 + two * cos_harmonic_sum(u, n)) - (Real.1 + two * cos_harmonic_sum(v, n)) = two * cos_harmonic_sum(u, n) - two * cos_harmonic_sum(v, n)

    dirichlet_kernel_fn(n, u) - dirichlet_kernel_fn(n, v) = two * cos_harmonic_sum(u, n) - two * cos_harmonic_sum(v, n)

    fn_partial_seq_cos_eq_sum(u, n)
    fn_partial_seq(seq_cos, n, u) = cos_harmonic_sum(u, n)
    fn_partial_seq_cos_eq_sum(v, n)
    fn_partial_seq(seq_cos, n, v) = cos_harmonic_sum(v, n)
    cos_harmonic_sum(u, n) = fn_partial_seq(seq_cos, n, u)
    cos_harmonic_sum(v, n) = fn_partial_seq(seq_cos, n, v)
    two * cos_harmonic_sum(u, n) - two * cos_harmonic_sum(v, n) = two * fn_partial_seq(seq_cos, n, u) - two * fn_partial_seq(seq_cos, n, v)

    dirichlet_kernel_fn(n, u) - dirichlet_kernel_fn(n, v) = two * fn_partial_seq(seq_cos, n, u) - two * fn_partial_seq(seq_cos, n, v)

    fn_partial_seq(seq_cos, n, u) = partial(seq_apply(seq_cos, u), n)
    fn_partial_seq(seq_cos, n, v) = partial(seq_apply(seq_cos, v), n)
    two * partial(seq_apply(seq_cos, u), n) - two * partial(seq_apply(seq_cos, v), n) = two * (partial(seq_apply(seq_cos, u), n) - partial(seq_apply(seq_cos, v), n))

    dirichlet_kernel_fn(n, u) - dirichlet_kernel_fn(n, v) = two * (partial(seq_apply(seq_cos, u), n) - partial(seq_apply(seq_cos, v), n))

    partial_diff_seq(seq_apply(seq_cos, u), seq_apply(seq_cos, v), n)
    partial(seq_apply(seq_cos, u), n) - partial(seq_apply(seq_cos, v), n) = partial(seq_diff(seq_apply(seq_cos, u), seq_apply(seq_cos, v)), n)

    dirichlet_kernel_fn(n, u) - dirichlet_kernel_fn(n, v) = two * partial(seq_diff(seq_apply(seq_cos, u), seq_apply(seq_cos, v)), n)

    forall(i: Nat) {
        if i < n {
            seq_diff(seq_apply(seq_cos, u), seq_apply(seq_cos, v), i) = seq_apply(seq_cos, u, i) - seq_apply(seq_cos, v, i)

            seq_apply(seq_cos, u, i) = seq_cos(i, u)
            seq_cos(i, u) = cos_nt(i.suc, u)
            seq_apply(seq_cos, u, i) = cos_nt(i.suc, u)
            seq_apply(seq_cos, v, i) = seq_cos(i, v)
            seq_cos(i, v) = cos_nt(i.suc, v)
            seq_apply(seq_cos, v, i) = cos_nt(i.suc, v)
            seq_diff(seq_apply(seq_cos, u), seq_apply(seq_cos, v), i) = cos_nt(i.suc, u) - cos_nt(i.suc, v)

            seq_diff_cos(u, v, i) = cos_nt(i.suc, u) - cos_nt(i.suc, v)
            seq_diff(seq_apply(seq_cos, u), seq_apply(seq_cos, v), i) = seq_diff_cos(u, v, i)
        }
    }
    partial_pointwise_eq(seq_diff(seq_apply(seq_cos, u), seq_apply(seq_cos, v)), seq_diff_cos(u, v), n)
    partial(seq_diff(seq_apply(seq_cos, u), seq_apply(seq_cos, v)), n) = partial(seq_diff_cos(u, v), n)
    dirichlet_kernel_fn(n, u) - dirichlet_kernel_fn(n, v) = two * partial(seq_diff_cos(u, v), n)
    (dirichlet_kernel_fn(n, u) - dirichlet_kernel_fn(n, v)).abs = (two * partial(seq_diff_cos(u, v), n)).abs
    mul_abs(two, partial(seq_diff_cos(u, v), n))
    (two * partial(seq_diff_cos(u, v), n)).abs = two.abs * partial(seq_diff_cos(u, v), n).abs
    two_positive
    two.is_positive
    pos_gt_zero(two)
    two > Real.0
    gt_zero_imp_pos(two)
    pos_imp_eq_abs(two)
    two.is_positive implies two.abs = two
    two.abs = two
    (two * partial(seq_diff_cos(u, v), n)).abs = two * partial(seq_diff_cos(u, v), n).abs
    (dirichlet_kernel_fn(n, u) - dirichlet_kernel_fn(n, v)).abs = two * partial(seq_diff_cos(u, v), n).abs

    partial_triangle_abs(seq_diff_cos(u, v), n)
    partial(seq_diff_cos(u, v), n).abs <= partial(seq_abs(seq_diff_cos(u, v)), n)
    two_positive
    two.is_positive
    pos_gt_zero(two)
    two > Real.0
    lt_imp_lte(Real.0, two)
    Real.0 <= two
    mul_le_mul_of_nonneg_left(partial(seq_diff_cos(u, v), n).abs, partial(seq_abs(seq_diff_cos(u, v)), n), two)
    two * partial(seq_diff_cos(u, v), n).abs <= two * partial(seq_abs(seq_diff_cos(u, v)), n)
    (dirichlet_kernel_fn(n, u) - dirichlet_kernel_fn(n, v)).abs <= two * partial(seq_abs(seq_diff_cos(u, v)), n)

    forall(i: Nat) {
        if i < n {
            seq_abs(seq_diff_cos(u, v), i) = (seq_diff_cos(u, v, i)).abs
            seq_diff_cos(u, v, i) = cos_nt(i.suc, u) - cos_nt(i.suc, v)
            seq_abs(seq_diff_cos(u, v), i) = (cos_nt(i.suc, u) - cos_nt(i.suc, v)).abs
            cos_nt_lipschitz(i.suc, u, v)
            (cos_nt(i.suc, u) - cos_nt(i.suc, v)).abs <= from_nat[Real](i.suc) * (u - v).abs
            seq_abs(seq_diff_cos(u, v), i) <= from_nat[Real](i.suc) * (u - v).abs
            seq_k_absdiff(u, v, i) = from_nat[Real](i.suc) * (u - v).abs
            seq_abs(seq_diff_cos(u, v), i) <= seq_k_absdiff(u, v, i)
        }
    }
    partial_lte_self(seq_abs(seq_diff_cos(u, v)), seq_k_absdiff(u, v), n)
    forall(i: Nat) { i < n implies seq_abs(seq_diff_cos(u, v), i) <= seq_k_absdiff(u, v, i) }
    partial(seq_abs(seq_diff_cos(u, v)), n) <= partial(seq_k_absdiff(u, v), n)
    two_positive
    two.is_positive
    pos_gt_zero(two)
    two > Real.0
    lt_imp_lte(Real.0, two)
    Real.0 <= two
    mul_le_mul_of_nonneg_left(partial(seq_abs(seq_diff_cos(u, v)), n), partial(seq_k_absdiff(u, v), n), two)
    two * partial(seq_abs(seq_diff_cos(u, v)), n) <= two * partial(seq_k_absdiff(u, v), n)
    lte_trans[Real]((dirichlet_kernel_fn(n, u) - dirichlet_kernel_fn(n, v)).abs, two * partial(seq_abs(seq_diff_cos(u, v)), n), two * partial(seq_k_absdiff(u, v), n))


    (dirichlet_kernel_fn(n, u) - dirichlet_kernel_fn(n, v)).abs <= two * partial(seq_k_absdiff(u, v), n)
    forall(i: Nat) {
        if i < n {
            mul_fn((u - v).abs, seq_k, i) = (u - v).abs * seq_k(i)
            seq_k(i) = from_nat[Real](i.suc)
            mul_fn((u - v).abs, seq_k, i) = (u - v).abs * from_nat[Real](i.suc)
            real_mul_comm((u - v).abs, from_nat[Real](i.suc))
            (u - v).abs * from_nat[Real](i.suc) = from_nat[Real](i.suc) * (u - v).abs
            mul_fn((u - v).abs, seq_k, i) = from_nat[Real](i.suc) * (u - v).abs
            seq_k_absdiff(u, v, i) = from_nat[Real](i.suc) * (u - v).abs
            mul_fn((u - v).abs, seq_k, i) = seq_k_absdiff(u, v, i)
        }
    }
    partial_pointwise_eq(mul_fn((u - v).abs, seq_k), seq_k_absdiff(u, v), n)
    partial(mul_fn((u - v).abs, seq_k), n) = partial(seq_k_absdiff(u, v), n)
    partial_scalar_mul((u - v).abs, seq_k, n)
    (u - v).abs * partial(seq_k, n) = partial(mul_fn((u - v).abs, seq_k), n)
    (u - v).abs * partial(seq_k, n) = partial(seq_k_absdiff(u, v), n)
    two * partial(seq_k_absdiff(u, v), n) = two * ((u - v).abs * partial(seq_k, n))
    real_mul_comm((u - v).abs, partial(seq_k, n))
    (u - v).abs * partial(seq_k, n) = partial(seq_k, n) * (u - v).abs
    two * ((u - v).abs * partial(seq_k, n)) = two * (partial(seq_k, n) * (u - v).abs)
    mul_assoc(two, partial(seq_k, n), (u - v).abs)
    two * (partial(seq_k, n) * (u - v).abs) = (two * partial(seq_k, n)) * (u - v).abs
    two * partial(seq_k_absdiff(u, v), n) = (two * partial(seq_k, n)) * (u - v).abs
    (dirichlet_kernel_fn(n, u) - dirichlet_kernel_fn(n, v)).abs <= (two * partial(seq_k, n)) * (u - v).abs

}

/// The Dirichlet kernel is bounded in absolute value by 1 + 2n.
theorem dirichlet_kernel_fn_abs_le(n: Nat, t: Real) {
    (dirichlet_kernel_fn(n, t)).abs <= Real.1 + two * from_nat[Real](n)
} by {
    dirichlet_kernel_fn(n, t) = dirichlet_kernel(t, n)
    dirichlet_kernel(t, n) = Real.1 + two * cos_harmonic_sum(t, n)
    dirichlet_kernel_fn(n, t) = Real.1 + two * cos_harmonic_sum(t, n)
    triangle_ineq(Real.1, two * cos_harmonic_sum(t, n))
    (Real.1 + two * cos_harmonic_sum(t, n)).abs <= Real.1.abs + (two * cos_harmonic_sum(t, n)).abs
    dirichlet_kernel_fn(n, t).abs <= Real.1.abs + (two * cos_harmonic_sum(t, n)).abs
    one_nonneg
    Real.0 <= Real.1
    abs_of_nonneg(Real.1)
    Real.1 >= Real.0 implies Real.1.abs = Real.1
    Real.1.abs = Real.1
    mul_abs(two, cos_harmonic_sum(t, n))
    (two * cos_harmonic_sum(t, n)).abs = two.abs * (cos_harmonic_sum(t, n)).abs
    two_positive
    two.is_positive
    pos_gt_zero(two)
    two > Real.0
    gt_zero_imp_pos(two)
    pos_imp_eq_abs(two)
    two.is_positive implies two.abs = two
    two.abs = two
    (two * cos_harmonic_sum(t, n)).abs = two * (cos_harmonic_sum(t, n)).abs
    dirichlet_kernel_fn(n, t).abs <= Real.1 + two * (cos_harmonic_sum(t, n)).abs
    fn_partial_seq_cos_eq_sum(t, n)
    fn_partial_seq(seq_cos, n, t) = cos_harmonic_sum(t, n)
    cos_harmonic_sum(t, n) = fn_partial_seq(seq_cos, n, t)
    fn_partial_seq(seq_cos, n, t) = partial(seq_apply(seq_cos, t), n)
    cos_harmonic_sum(t, n) = partial(seq_apply(seq_cos, t), n)
    (cos_harmonic_sum(t, n)).abs = partial(seq_apply(seq_cos, t), n).abs
    partial_triangle_abs(seq_apply(seq_cos, t), n)
    partial(seq_apply(seq_cos, t), n).abs <= partial(seq_abs(seq_apply(seq_cos, t)), n)
    (cos_harmonic_sum(t, n)).abs <= partial(seq_abs(seq_apply(seq_cos, t)), n)
    forall(i: Nat) {
        if i < n {
            seq_abs(seq_apply(seq_cos, t), i) = (seq_apply(seq_cos, t, i)).abs
            seq_apply(seq_cos, t, i) = seq_cos(i, t)
            seq_cos(i, t) = cos_nt(i.suc, t)
            seq_apply(seq_cos, t, i) = cos_nt(i.suc, t)
            seq_abs(seq_apply(seq_cos, t), i) = (cos_nt(i.suc, t)).abs
            cos_nt_abs_le_one(i.suc, t)
            (cos_nt(i.suc, t)).abs <= Real.1
            seq_abs(seq_apply(seq_cos, t), i) <= Real.1
            seq_one(i) = Real.1
            seq_abs(seq_apply(seq_cos, t), i) <= seq_one(i)
        }
    }
    partial_lte_self(seq_abs(seq_apply(seq_cos, t)), seq_one, n)
    forall(i: Nat) { i < n implies seq_abs(seq_apply(seq_cos, t), i) <= seq_one(i) }
    partial(seq_abs(seq_apply(seq_cos, t)), n) <= partial(seq_one, n)
    partial_seq_one(n)
    partial(seq_one, n) = from_nat[Real](n)
    partial(seq_abs(seq_apply(seq_cos, t)), n) <= from_nat[Real](n)
    lte_trans[Real](partial(seq_apply(seq_cos, t), n).abs, partial(seq_abs(seq_apply(seq_cos, t)), n), from_nat[Real](n))

    partial(seq_apply(seq_cos, t), n).abs <= from_nat[Real](n)
    (cos_harmonic_sum(t, n)).abs = partial(seq_apply(seq_cos, t), n).abs
    lte_trans[Real]((cos_harmonic_sum(t, n)).abs, partial(seq_apply(seq_cos, t), n).abs, from_nat[Real](n))
    (cos_harmonic_sum(t, n)).abs <= from_nat[Real](n)
    two_positive
    two.is_positive
    pos_gt_zero(two)
    two > Real.0
    lt_imp_lte(Real.0, two)
    Real.0 <= two
    mul_le_mul_of_nonneg_left((cos_harmonic_sum(t, n)).abs, from_nat[Real](n), two)
    two * (cos_harmonic_sum(t, n)).abs <= two * from_nat[Real](n)
    add_le_add(Real.1, Real.1, two * (cos_harmonic_sum(t, n)).abs, two * from_nat[Real](n))
    Real.1 + two * (cos_harmonic_sum(t, n)).abs <= Real.1 + two * from_nat[Real](n)
    lte_trans[Real](dirichlet_kernel_fn(n, t).abs, Real.1 + two * (cos_harmonic_sum(t, n)).abs, Real.1 + two * from_nat[Real](n))

    (dirichlet_kernel_fn(n, t)).abs <= Real.1 + two * from_nat[Real](n)
}


/// The partial sum of a sequence of continuous functions is continuous.
theorem continuous_fn_partial_seq(seq: Nat -> (Real -> Real), n: Nat) {
    (forall(i: Nat) { i < n implies continuous(seq(i)) })
    implies continuous(fn_partial_seq(seq, n))
} by {
    define q(m: Nat) -> Bool {
        (forall(i: Nat) { i < m implies continuous(seq(i)) })
        implies continuous(fn_partial_seq(seq, m))
    }
    forall(x: Real) {
        partial_zero(seq_apply(seq, x))
        partial(seq_apply(seq, x), Nat.0) = Real.0
        fn_partial_seq(seq, Nat.0, x) = partial(seq_apply(seq, x), Nat.0)
        fn_partial_seq(seq, Nat.0, x) = Real.0
        constant[Real, Real](Real.0, x) = Real.0
        fn_partial_seq(seq, Nat.0, x) = constant[Real, Real](Real.0, x)
    }
    function_extensionality(fn_partial_seq(seq, Nat.0), constant[Real, Real](Real.0))
    fn_partial_seq(seq, Nat.0) = constant[Real, Real](Real.0)
    constant_function_is_continuous(Real.0)
    continuous(constant[Real, Real](Real.0))
    continuous(fn_partial_seq(seq, Nat.0))
    q(Nat.0)
    forall(m: Nat) {
        if q(m) {
            if forall(i: Nat) { i < m.suc implies continuous(seq(i)) } {
                lt_suc(m)
                m < m.suc
                forall(i: Nat) {
                    if i < m {
                        lt_trans(i, m, m.suc)
                        i < m and m < m.suc implies i < m.suc
                        i < m.suc
                        continuous(seq(i))
                    }
                }
                forall(i: Nat) { i < m implies continuous(seq(i)) }
                q(m) = ((forall(i: Nat) { i < m implies continuous(seq(i)) }) implies continuous(fn_partial_seq(seq, m)))
                (forall(i: Nat) { i < m implies continuous(seq(i)) }) implies continuous(fn_partial_seq(seq, m))
                continuous(fn_partial_seq(seq, m))
                continuous(seq(m))
                continuous_pointwise_add(fn_partial_seq(seq, m), seq(m))
                continuous(fn_partial_seq(seq, m)) and continuous(seq(m)) implies continuous(pointwise_add(fn_partial_seq(seq, m), seq(m)))

                continuous(pointwise_add(fn_partial_seq(seq, m), seq(m)))
                forall(x: Real) {
                    partial_suc(seq_apply(seq, x), m)
                    partial(seq_apply(seq, x), m.suc) = partial(seq_apply(seq, x), m) + seq_apply(seq, x, m)
                    fn_partial_seq(seq, m.suc, x) = partial(seq_apply(seq, x), m.suc)
                    fn_partial_seq(seq, m.suc, x) = partial(seq_apply(seq, x), m) + seq_apply(seq, x, m)
                    fn_partial_seq(seq, m, x) = partial(seq_apply(seq, x), m)
                    fn_partial_seq(seq, m.suc, x) = fn_partial_seq(seq, m, x) + seq_apply(seq, x, m)
                    seq_apply(seq, x, m) = seq(m, x)
                    fn_partial_seq(seq, m.suc, x) = fn_partial_seq(seq, m, x) + seq(m, x)
                    pointwise_add_apply(fn_partial_seq(seq, m), seq(m), x)
                    pointwise_add(fn_partial_seq(seq, m), seq(m), x) = fn_partial_seq(seq, m, x) + seq(m, x)

                    fn_partial_seq(seq, m.suc, x) = pointwise_add(fn_partial_seq(seq, m), seq(m), x)
                }
                function_extensionality(fn_partial_seq(seq, m.suc), pointwise_add(fn_partial_seq(seq, m), seq(m)))

                fn_partial_seq(seq, m.suc) = pointwise_add(fn_partial_seq(seq, m), seq(m))
                continuous(fn_partial_seq(seq, m.suc))
                q(m.suc)
            }
        }
    }
    alt_induction(q)
    (q(Nat.0) and forall(j: Nat) { q(j) implies q(j.suc) }) implies forall(j: Nat) { q(j) }
    q(Nat.0) and forall(j: Nat) { q(j) implies q(j.suc) }
    forall(j: Nat) { q(j) }
    q(n)
}

/// Each term of the sine-over-k sequence is continuous.
theorem seq_sin_over_k_continuous(i: Nat) {
    continuous(seq_sin_over_k(i))
} by {
    constant_function_is_continuous(Real.1 / from_nat[Real](i.suc))
    continuous(constant[Real, Real](Real.1 / from_nat[Real](i.suc)))
    sin_nt_continuous(i.suc)
    continuous(sin_nt(i.suc))
    continuous_pointwise_mul(constant[Real, Real](Real.1 / from_nat[Real](i.suc)), sin_nt(i.suc))
    continuous(constant[Real, Real](Real.1 / from_nat[Real](i.suc))) and continuous(sin_nt(i.suc)) implies continuous(pointwise_mul(constant[Real, Real](Real.1 / from_nat[Real](i.suc)), sin_nt(i.suc)))

    continuous(pointwise_mul(constant[Real, Real](Real.1 / from_nat[Real](i.suc)), sin_nt(i.suc)))
    seq_sin_over_k(i) = pointwise_mul(constant[Real, Real](Real.1 / from_nat[Real](i.suc)), sin_nt(i.suc))
    continuous(seq_sin_over_k(i))
}

/// The Dirichlet-kernel antiderivative is continuous.
theorem g_dirichlet_continuous(n: Nat) {
    continuous(g_dirichlet(n))
} by {
    forall(i: Nat) {
        if i < n {
            seq_sin_over_k_continuous(i)
            continuous(seq_sin_over_k(i))
        }
    }
    continuous_fn_partial_seq(seq_sin_over_k, n)
    forall(i: Nat) { i < n implies continuous(seq_sin_over_k(i)) }
    continuous(fn_partial_seq(seq_sin_over_k, n))
    constant_function_is_continuous(two)
    continuous(constant[Real, Real](two))
    continuous_pointwise_mul(constant[Real, Real](two), fn_partial_seq(seq_sin_over_k, n))
    continuous(constant[Real, Real](two)) and continuous(fn_partial_seq(seq_sin_over_k, n)) implies continuous(pointwise_mul(constant[Real, Real](two), fn_partial_seq(seq_sin_over_k, n)))

    continuous(pointwise_mul(constant[Real, Real](two), fn_partial_seq(seq_sin_over_k, n)))
    identity_function_is_continuous
    continuous(identity_fn[Real])
    continuous_pointwise_add(identity_fn[Real], pointwise_mul(constant[Real, Real](two), fn_partial_seq(seq_sin_over_k, n)))

    continuous(identity_fn[Real]) and continuous(pointwise_mul(constant[Real, Real](two), fn_partial_seq(seq_sin_over_k, n))) implies continuous(pointwise_add(identity_fn[Real],   pointwise_mul(constant[Real, Real](two), fn_partial_seq(seq_sin_over_k, n))))

    continuous(pointwise_add(identity_fn[Real], pointwise_mul(constant[Real, Real](two), fn_partial_seq(seq_sin_over_k, n))))

    g_dirichlet(n) = pointwise_add(identity_fn[Real], pointwise_mul(constant[Real, Real](two), fn_partial_seq(seq_sin_over_k, n)))

    continuous(g_dirichlet(n))
}

/// The partial sum of the positive-integer sequence is nonnegative.
theorem partial_seq_k_nonneg(n: Nat) {
    Real.0 <= partial(seq_k, n)
} by {
    define q(m: Nat) -> Bool {
        Real.0 <= partial(seq_k, m)
    }
    partial_zero(seq_k)
    partial(seq_k, Nat.0) = Real.0
    lte_self(Real.0)
    Real.0 <= Real.0
    Real.0 <= partial(seq_k, Nat.0)
    q(Nat.0)
    forall(m: Nat) {
        if q(m) {
            partial_suc(seq_k, m)
            partial(seq_k, m.suc) = partial(seq_k, m) + seq_k(m)
            seq_k(m) = from_nat[Real](m.suc)
            partial(seq_k, m.suc) = partial(seq_k, m) + from_nat[Real](m.suc)
            Real.0 <= partial(seq_k, m)
            from_nat_nonneg_real(m.suc)
            Real.0 <= from_nat[Real](m.suc)
            add_le_add(Real.0, partial(seq_k, m), Real.0, from_nat[Real](m.suc))
            Real.0 + Real.0 <= partial(seq_k, m) + from_nat[Real](m.suc)
            Real.0 + Real.0 = Real.0
            Real.0 <= partial(seq_k, m) + from_nat[Real](m.suc)
            Real.0 <= partial(seq_k, m.suc)
            q(m.suc)
        }
    }
    alt_induction(q)
    (q(Nat.0) and forall(j: Nat) { q(j) implies q(j.suc) }) implies forall(j: Nat) { q(j) }
    q(Nat.0) and forall(j: Nat) { q(j) implies q(j.suc) }
    forall(j: Nat) { q(j) }
    q(n)
}

/// The Dirichlet kernel is integrable on [0, 2*pi].
theorem dirichlet_kernel_fn_integrable(n: Nat) {
    is_integrable(dirichlet_kernel_fn(n), Real.0, two_pi)
} by {
        two_pi_nonneg
        Real.0 <= two_pi
        partial_seq_k_nonneg(n)
        Real.0 <= partial(seq_k, n)
        two_positive
        two.is_positive
        pos_gt_zero(two)
        two > Real.0
        lt_imp_lte(Real.0, two)
        Real.0 <= two
        two >= Real.0
        partial(seq_k, n) >= Real.0
        mul_nonneg(two, partial(seq_k, n))
        two >= Real.0 and partial(seq_k, n) >= Real.0 implies two * partial(seq_k, n) >= Real.0
        two * partial(seq_k, n) >= Real.0
        Real.0 <= two * partial(seq_k, n)
        g_dirichlet_continuous(n)
        continuous(g_dirichlet(n))
        g_dirichlet_has_derivative(n)
        is_derivative_fn(g_dirichlet(n), dirichlet_kernel_fn(n))
        forall(u: Real, v: Real) {
            if interval_contains(Real.0, two_pi, u) and interval_contains(Real.0, two_pi, v) {
                dirichlet_kernel_fn_lipschitz(n, u, v)
                (dirichlet_kernel_fn(n, u) - dirichlet_kernel_fn(n, v)).abs <= (two * partial(seq_k, n)) * (u - v).abs

            }
        }
        forall(t: Real) {
            if interval_contains(Real.0, two_pi, t) {
                dirichlet_kernel_fn_abs_le(n, t)
                (dirichlet_kernel_fn(n, t)).abs <= Real.1 + two * from_nat[Real](n)
                abs_le_imp_lower(dirichlet_kernel_fn(n, t), Real.1 + two * from_nat[Real](n))
                -(Real.1 + two * from_nat[Real](n)) <= dirichlet_kernel_fn(n, t)
            }
        }
        forall(t: Real) {
            if interval_contains(Real.0, two_pi, t) {
                dirichlet_kernel_fn_abs_le(n, t)
                (dirichlet_kernel_fn(n, t)).abs <= Real.1 + two * from_nat[Real](n)
                abs_le_imp_upper(dirichlet_kernel_fn(n, t), Real.1 + two * from_nat[Real](n))
                dirichlet_kernel_fn(n, t) <= Real.1 + two * from_nat[Real](n)
            }
        }
        Real.0 <= two_pi
        (Real.0 <= two_pi) and Real.0 <= two * partial(seq_k, n)
        ((Real.0 <= two_pi) and Real.0 <= two * partial(seq_k, n)) and continuous(g_dirichlet(n))
        (((Real.0 <= two_pi) and Real.0 <= two * partial(seq_k, n)) and continuous(g_dirichlet(n))) and is_derivative_fn(g_dirichlet(n), dirichlet_kernel_fn(n))

        ((((Real.0 <= two_pi) and Real.0 <= two * partial(seq_k, n)) and continuous(g_dirichlet(n))) and is_derivative_fn(g_dirichlet(n), dirichlet_kernel_fn(n))) and forall(u: Real, v: Real) {

                interval_contains(Real.0, two_pi, u) and interval_contains(Real.0, two_pi, v)
                implies (dirichlet_kernel_fn(n, u) - dirichlet_kernel_fn(n, v)).abs <= (two * partial(seq_k, n)) * (u - v).abs

            }
        (((((Real.0 <= two_pi) and Real.0 <= two * partial(seq_k, n)) and continuous(g_dirichlet(n))) and is_derivative_fn(g_dirichlet(n), dirichlet_kernel_fn(n))) and forall(u: Real, v: Real) {

                interval_contains(Real.0, two_pi, u) and interval_contains(Real.0, two_pi, v)
                implies (dirichlet_kernel_fn(n, u) - dirichlet_kernel_fn(n, v)).abs <= (two * partial(seq_k, n)) * (u - v).abs

            }) and forall(t: Real) {
                interval_contains(Real.0, two_pi, t) implies -(Real.1 + two * from_nat[Real](n)) <= dirichlet_kernel_fn(n, t)
            }
        ((((((Real.0 <= two_pi) and Real.0 <= two * partial(seq_k, n)) and continuous(g_dirichlet(n))) and is_derivative_fn(g_dirichlet(n), dirichlet_kernel_fn(n))) and forall(u: Real, v: Real) {

                interval_contains(Real.0, two_pi, u) and interval_contains(Real.0, two_pi, v)
                implies (dirichlet_kernel_fn(n, u) - dirichlet_kernel_fn(n, v)).abs <= (two * partial(seq_k, n)) * (u - v).abs

            }) and forall(t: Real) {
                interval_contains(Real.0, two_pi, t) implies -(Real.1 + two * from_nat[Real](n)) <= dirichlet_kernel_fn(n, t)
            }) and forall(t: Real) {
                interval_contains(Real.0, two_pi, t) implies dirichlet_kernel_fn(n, t) <= Real.1 + two * from_nat[Real](n)
            }
        if ((((((Real.0 <= two_pi) and Real.0 <= two * partial(seq_k, n)) and continuous(g_dirichlet(n))) and is_derivative_fn(g_dirichlet(n), dirichlet_kernel_fn(n))) and forall(u: Real, v: Real) {

                interval_contains(Real.0, two_pi, u) and interval_contains(Real.0, two_pi, v)
                implies (dirichlet_kernel_fn(n, u) - dirichlet_kernel_fn(n, v)).abs <= (two * partial(seq_k, n)) * (u - v).abs

            }) and forall(t: Real) {
                interval_contains(Real.0, two_pi, t) implies -(Real.1 + two * from_nat[Real](n)) <= dirichlet_kernel_fn(n, t)
            }) and forall(t: Real) {
                interval_contains(Real.0, two_pi, t) implies dirichlet_kernel_fn(n, t) <= Real.1 + two * from_nat[Real](n)
            } {
            fn_integrable_gen(dirichlet_kernel_fn(n), g_dirichlet(n), Real.0, two_pi, two * partial(seq_k, n), -(Real.1 + two * from_nat[Real](n)), Real.1 + two * from_nat[Real](n))

            is_integrable(dirichlet_kernel_fn(n), Real.0, two_pi)
        }
}

/// The partial sum of the sine-over-k sequence vanishes at the period.
theorem partial_seq_sin_over_k_at_two_pi(n: Nat) {
    partial(seq_apply(seq_sin_over_k, two_pi), n) = Real.0
} by {
    forall(i: Nat) {
        if i < n {
            seq_apply(seq_sin_over_k, two_pi, i) = seq_sin_over_k(i, two_pi)
            seq_sin_over_k(i, two_pi) = constant[Real, Real](Real.1 / from_nat[Real](i.suc), two_pi) * sin_nt(i.suc, two_pi)

            pointwise_mul_apply(constant[Real, Real](Real.1 / from_nat[Real](i.suc)), sin_nt(i.suc), two_pi)
            pointwise_mul(constant[Real, Real](Real.1 / from_nat[Real](i.suc)), sin_nt(i.suc), two_pi) = constant[Real, Real](Real.1 / from_nat[Real](i.suc), two_pi) * sin_nt(i.suc, two_pi)

            constant[Real, Real](Real.1 / from_nat[Real](i.suc), two_pi) = Real.1 / from_nat[Real](i.suc)
            pointwise_mul(constant[Real, Real](Real.1 / from_nat[Real](i.suc)), sin_nt(i.suc), two_pi) = (Real.1 / from_nat[Real](i.suc)) * sin_nt(i.suc, two_pi)

            seq_sin_over_k(i, two_pi) = (Real.1 / from_nat[Real](i.suc)) * sin_nt(i.suc, two_pi)
            seq_apply(seq_sin_over_k, two_pi, i) = (Real.1 / from_nat[Real](i.suc)) * sin_nt(i.suc, two_pi)
            sin_nt_two_pi_zero(i.suc)
            sin_nt(i.suc, two_pi) = Real.0
            (Real.1 / from_nat[Real](i.suc)) * sin_nt(i.suc, two_pi) = Real.0
            seq_apply(seq_sin_over_k, two_pi, i) = Real.0
        }
    }
    partial_seq_zero(seq_apply(seq_sin_over_k, two_pi), n)
    forall(i: Nat) { i < n implies seq_apply(seq_sin_over_k, two_pi, i) = Real.0 }
    partial(seq_apply(seq_sin_over_k, two_pi), n) = Real.0
}

/// The partial sum of the sine-over-k sequence vanishes at zero.
theorem partial_seq_sin_over_k_at_zero(n: Nat) {
    partial(seq_apply(seq_sin_over_k, Real.0), n) = Real.0
} by {
    forall(i: Nat) {
        if i < n {
            seq_apply(seq_sin_over_k, Real.0, i) = seq_sin_over_k(i, Real.0)
            pointwise_mul_apply(constant[Real, Real](Real.1 / from_nat[Real](i.suc)), sin_nt(i.suc), Real.0)
            pointwise_mul(constant[Real, Real](Real.1 / from_nat[Real](i.suc)), sin_nt(i.suc), Real.0) = constant[Real, Real](Real.1 / from_nat[Real](i.suc), Real.0) * sin_nt(i.suc, Real.0)

            constant[Real, Real](Real.1 / from_nat[Real](i.suc), Real.0) = Real.1 / from_nat[Real](i.suc)
            pointwise_mul(constant[Real, Real](Real.1 / from_nat[Real](i.suc)), sin_nt(i.suc), Real.0) = (Real.1 / from_nat[Real](i.suc)) * sin_nt(i.suc, Real.0)

            seq_sin_over_k(i, Real.0) = (Real.1 / from_nat[Real](i.suc)) * sin_nt(i.suc, Real.0)
            seq_apply(seq_sin_over_k, Real.0, i) = (Real.1 / from_nat[Real](i.suc)) * sin_nt(i.suc, Real.0)
            sin_nt_at_zero(i.suc)
            sin_nt(i.suc, Real.0) = Real.0
            (Real.1 / from_nat[Real](i.suc)) * sin_nt(i.suc, Real.0) = Real.0
            seq_apply(seq_sin_over_k, Real.0, i) = Real.0
        }
    }
    partial_seq_zero(seq_apply(seq_sin_over_k, Real.0), n)
    forall(i: Nat) { i < n implies seq_apply(seq_sin_over_k, Real.0, i) = Real.0 }
    partial(seq_apply(seq_sin_over_k, Real.0), n) = Real.0
}

/// The integral of the Dirichlet kernel over [0, 2*pi] is 2*pi.
theorem integral_dirichlet_kernel_two_pi(n: Nat) {
    integral(dirichlet_kernel_fn(n), Real.0, two_pi) = two_pi
} by {
        two_pi_nonneg
        Real.0 <= two_pi
        g_dirichlet_continuous(n)
        continuous(g_dirichlet(n))
        g_dirichlet_has_derivative(n)
        is_derivative_fn(g_dirichlet(n), dirichlet_kernel_fn(n))
        dirichlet_kernel_fn_integrable(n)
        is_integrable(dirichlet_kernel_fn(n), Real.0, two_pi)
        forall(t: Real) {
            if interval_contains(Real.0, two_pi, t) {
                dirichlet_kernel_fn_abs_le(n, t)
                (dirichlet_kernel_fn(n, t)).abs <= Real.1 + two * from_nat[Real](n)
                abs_le_imp_lower(dirichlet_kernel_fn(n, t), Real.1 + two * from_nat[Real](n))
                -(Real.1 + two * from_nat[Real](n)) <= dirichlet_kernel_fn(n, t)
            }
        }
        forall(t: Real) {
            if interval_contains(Real.0, two_pi, t) {
                dirichlet_kernel_fn_abs_le(n, t)
                (dirichlet_kernel_fn(n, t)).abs <= Real.1 + two * from_nat[Real](n)
                abs_le_imp_upper(dirichlet_kernel_fn(n, t), Real.1 + two * from_nat[Real](n))
                dirichlet_kernel_fn(n, t) <= Real.1 + two * from_nat[Real](n)
            }
        }
        Real.0 <= two_pi
        (Real.0 <= two_pi) and continuous(g_dirichlet(n))
        ((Real.0 <= two_pi) and continuous(g_dirichlet(n))) and is_derivative_fn(g_dirichlet(n), dirichlet_kernel_fn(n))
        (((Real.0 <= two_pi) and continuous(g_dirichlet(n))) and is_derivative_fn(g_dirichlet(n), dirichlet_kernel_fn(n))) and is_integrable(dirichlet_kernel_fn(n), Real.0, two_pi)

        ((((Real.0 <= two_pi) and continuous(g_dirichlet(n))) and is_derivative_fn(g_dirichlet(n), dirichlet_kernel_fn(n))) and is_integrable(dirichlet_kernel_fn(n), Real.0, two_pi)) and forall(t: Real) {

                interval_contains(Real.0, two_pi, t) implies -(Real.1 + two * from_nat[Real](n)) <= dirichlet_kernel_fn(n, t)
            }
        (((((Real.0 <= two_pi) and continuous(g_dirichlet(n))) and is_derivative_fn(g_dirichlet(n), dirichlet_kernel_fn(n))) and is_integrable(dirichlet_kernel_fn(n), Real.0, two_pi)) and forall(t: Real) {

                interval_contains(Real.0, two_pi, t) implies -(Real.1 + two * from_nat[Real](n)) <= dirichlet_kernel_fn(n, t)
            }) and forall(t: Real) {
                interval_contains(Real.0, two_pi, t) implies dirichlet_kernel_fn(n, t) <= Real.1 + two * from_nat[Real](n)
            }
        if (((((Real.0 <= two_pi) and continuous(g_dirichlet(n))) and is_derivative_fn(g_dirichlet(n), dirichlet_kernel_fn(n))) and is_integrable(dirichlet_kernel_fn(n), Real.0, two_pi)) and forall(t: Real) {

                interval_contains(Real.0, two_pi, t) implies -(Real.1 + two * from_nat[Real](n)) <= dirichlet_kernel_fn(n, t)
            }) and forall(t: Real) {
                interval_contains(Real.0, two_pi, t) implies dirichlet_kernel_fn(n, t) <= Real.1 + two * from_nat[Real](n)
            } {
            ftc2_general(dirichlet_kernel_fn(n), g_dirichlet(n), Real.0, two_pi, -(Real.1 + two * from_nat[Real](n)), Real.1 + two * from_nat[Real](n))

            integral(dirichlet_kernel_fn(n), Real.0, two_pi) = g_dirichlet(n)(two_pi) - g_dirichlet(n)(Real.0)
            pointwise_add_apply(identity_fn[Real], pointwise_mul(constant[Real, Real](two), fn_partial_seq(seq_sin_over_k, n)), two_pi)

            pointwise_add(identity_fn[Real], pointwise_mul(constant[Real, Real](two), fn_partial_seq(seq_sin_over_k, n)), two_pi) = identity_fn[Real](two_pi) +


                    pointwise_mul(constant[Real, Real](two), fn_partial_seq(seq_sin_over_k, n), two_pi)
            identity_fn[Real](two_pi) = two_pi
            pointwise_mul_apply(constant[Real, Real](two), fn_partial_seq(seq_sin_over_k, n), two_pi)
            pointwise_mul(constant[Real, Real](two), fn_partial_seq(seq_sin_over_k, n), two_pi) = constant[Real, Real](two, two_pi) * fn_partial_seq(seq_sin_over_k, n, two_pi)

            constant[Real, Real](two, two_pi) = two
            pointwise_mul(constant[Real, Real](two), fn_partial_seq(seq_sin_over_k, n), two_pi) = two * fn_partial_seq(seq_sin_over_k, n, two_pi)

            pointwise_add(identity_fn[Real], pointwise_mul(constant[Real, Real](two), fn_partial_seq(seq_sin_over_k, n)), two_pi) = two_pi + two * fn_partial_seq(seq_sin_over_k, n, two_pi)


            g_dirichlet(n)(two_pi) = two_pi + two * fn_partial_seq(seq_sin_over_k, n, two_pi)
            fn_partial_seq(seq_sin_over_k, n, two_pi) = partial(seq_apply(seq_sin_over_k, two_pi), n)
            partial_seq_sin_over_k_at_two_pi(n)
            partial(seq_apply(seq_sin_over_k, two_pi), n) = Real.0
            fn_partial_seq(seq_sin_over_k, n, two_pi) = Real.0
            two * fn_partial_seq(seq_sin_over_k, n, two_pi) = Real.0
            g_dirichlet(n)(two_pi) = two_pi
            pointwise_add_apply(identity_fn[Real], pointwise_mul(constant[Real, Real](two), fn_partial_seq(seq_sin_over_k, n)), Real.0)

            pointwise_add(identity_fn[Real], pointwise_mul(constant[Real, Real](two), fn_partial_seq(seq_sin_over_k, n)), Real.0) = identity_fn[Real](Real.0) +


                    pointwise_mul(constant[Real, Real](two), fn_partial_seq(seq_sin_over_k, n), Real.0)
            identity_fn[Real](Real.0) = Real.0
            pointwise_mul_apply(constant[Real, Real](two), fn_partial_seq(seq_sin_over_k, n), Real.0)
            pointwise_mul(constant[Real, Real](two), fn_partial_seq(seq_sin_over_k, n), Real.0) = constant[Real, Real](two, Real.0) * fn_partial_seq(seq_sin_over_k, n, Real.0)

            constant[Real, Real](two, Real.0) = two
            pointwise_mul(constant[Real, Real](two), fn_partial_seq(seq_sin_over_k, n), Real.0) = two * fn_partial_seq(seq_sin_over_k, n, Real.0)

            pointwise_add(identity_fn[Real], pointwise_mul(constant[Real, Real](two), fn_partial_seq(seq_sin_over_k, n)), Real.0) = Real.0 + two * fn_partial_seq(seq_sin_over_k, n, Real.0)


            g_dirichlet(n)(Real.0) = Real.0 + two * fn_partial_seq(seq_sin_over_k, n, Real.0)
            fn_partial_seq(seq_sin_over_k, n, Real.0) = partial(seq_apply(seq_sin_over_k, Real.0), n)
            partial_seq_sin_over_k_at_zero(n)
            partial(seq_apply(seq_sin_over_k, Real.0), n) = Real.0
            fn_partial_seq(seq_sin_over_k, n, Real.0) = Real.0
            two * fn_partial_seq(seq_sin_over_k, n, Real.0) = Real.0
            g_dirichlet(n)(Real.0) = Real.0
            g_dirichlet(n)(two_pi) - g_dirichlet(n)(Real.0) = two_pi - Real.0
            two_pi - Real.0 = two_pi
            integral(dirichlet_kernel_fn(n), Real.0, two_pi) = two_pi
        }
}

/// The normalized integral mean of f over [0, 2*pi]:
/// (1 / 2*pi) * integral(f, 0, 2*pi).
define mean_integral(f: Real -> Real) -> Real {
    integral(f, Real.0, two_pi) / two_pi
}

/// The mean of the Dirichlet kernel over [0, 2*pi] is one.
theorem dirichlet_kernel_mean_one(n: Nat) {
    mean_integral(dirichlet_kernel_fn(n)) = Real.1
} by {
    mean_integral(dirichlet_kernel_fn(n)) = integral(dirichlet_kernel_fn(n), Real.0, two_pi) / two_pi
    integral_dirichlet_kernel_two_pi(n)
    integral(dirichlet_kernel_fn(n), Real.0, two_pi) = two_pi
    mean_integral(dirichlet_kernel_fn(n)) = two_pi / two_pi
    two_pi_pos
    Real.0 < two_pi
    lt_imp_ne(Real.0, two_pi)
    Real.0 < two_pi implies Real.0 != two_pi
    Real.0 != two_pi
    two_pi != Real.0
    div_mul_cancel_left(two_pi, Real.1)
    two_pi != Real.0 implies (two_pi * Real.1) / two_pi = Real.1
    (two_pi * Real.1) / two_pi = Real.1
    two_pi * Real.1 = two_pi
    two_pi / two_pi = Real.1
    mean_integral(dirichlet_kernel_fn(n)) = Real.1
}

// ---------------------------------------------------------------------------
// The Fourier coefficients of the harmonics themselves
// ---------------------------------------------------------------------------

/// The integral of sine over [0, 2*pi] restated with the two_pi constant.
theorem integral_sin_zero_period {
    integral(Real.sin, Real.0, two_pi) = Real.0
} by {
    two_pi = pi + pi
    pi + pi = two * pi
    add_self_two(pi)
    two_pi = two * pi
    integral_sin_zero_two_pi
    integral(Real.sin, Real.0, two * pi) = Real.0
    integral(Real.sin, Real.0, two_pi) = integral(Real.sin, Real.0, two * pi)
    integral(Real.sin, Real.0, two_pi) = Real.0
}

/// The integral of cosine over [0, 2*pi] restated with the two_pi constant.
theorem integral_cos_zero_period {
    integral(Real.cos, Real.0, two_pi) = Real.0
} by {
    two_pi = pi + pi
    pi + pi = two * pi
    add_self_two(pi)
    two_pi = two * pi
    integral_cos_zero_two_pi
    integral(Real.cos, Real.0, two * pi) = Real.0
    integral(Real.cos, Real.0, two_pi) = integral(Real.cos, Real.0, two * pi)
    integral(Real.cos, Real.0, two_pi) = Real.0
}

/// The a_0 coefficient of f(x) = x.sin is zero.
theorem fourier_cos_coeff_sin_zero {
    fourier_cos_coeff(Real.sin, Nat.0) = Real.0
} by {
    forall(x: Real) {
        fourier_cos_integrand(Real.sin, Nat.0, x) = x.sin * cos_nt(Nat.0, x)
        cos_nt_zero_harmonic(x)
        cos_nt(Nat.0, x) = Real.1
        x.sin * cos_nt(Nat.0, x) = x.sin * Real.1
        x.sin * Real.1 = x.sin
        fourier_cos_integrand(Real.sin, Nat.0, x) = x.sin
    }
    function_extensionality(fourier_cos_integrand(Real.sin, Nat.0), Real.sin)
    fourier_cos_integrand(Real.sin, Nat.0) = Real.sin
    integral(fourier_cos_integrand(Real.sin, Nat.0), Real.0, two_pi) = integral(Real.sin, Real.0, two_pi)
    integral_sin_zero_period
    integral(Real.sin, Real.0, two_pi) = Real.0
    integral(fourier_cos_integrand(Real.sin, Nat.0), Real.0, two_pi) = Real.0
    fourier_cos_coeff(Real.sin, Nat.0) = integral(fourier_cos_integrand(Real.sin, Nat.0), Real.0, two_pi) / pi
    fourier_cos_coeff(Real.sin, Nat.0) = Real.0 / pi
    Real.0 / pi = Real.0
    fourier_cos_coeff(Real.sin, Nat.0) = Real.0
}

/// The a_1 coefficient of f(x) = x.sin is zero (Real.sin is orthogonal to Real.cos).
theorem fourier_cos_coeff_sin_one {
    fourier_cos_coeff(Real.sin, Nat.1) = Real.0
} by {
    forall(x: Real) {
        fourier_cos_integrand(Real.sin, Nat.1, x) = x.sin * cos_nt(Nat.1, x)
        cos_nt(Nat.1, x) = (from_nat[Real](Nat.1) * x).cos
        from_nat[Real](Nat.1) = Real.1
        from_nat[Real](Nat.1) * x = x
        cos_nt(Nat.1, x) = x.cos
        x.sin * cos_nt(Nat.1, x) = x.sin * x.cos
        fourier_cos_integrand(Real.sin, Nat.1, x) = x.sin * x.cos
        fn_sincos(x) = x.sin * x.cos
        fourier_cos_integrand(Real.sin, Nat.1, x) = fn_sincos(x)
    }
    function_extensionality(fourier_cos_integrand(Real.sin, Nat.1), fn_sincos)
    fourier_cos_integrand(Real.sin, Nat.1) = fn_sincos
    integral(fourier_cos_integrand(Real.sin, Nat.1), Real.0, two_pi) = integral(fn_sincos, Real.0, two_pi)
    integral_sin_mul_cos_zero
    integral(fn_sincos, Real.0, two_pi) = Real.0
    integral(fourier_cos_integrand(Real.sin, Nat.1), Real.0, two_pi) = Real.0
    fourier_cos_coeff(Real.sin, Nat.1) = integral(fourier_cos_integrand(Real.sin, Nat.1), Real.0, two_pi) / pi
    fourier_cos_coeff(Real.sin, Nat.1) = Real.0 / pi
    Real.0 / pi = Real.0
    fourier_cos_coeff(Real.sin, Nat.1) = Real.0
}

/// The b_1 coefficient of f(x) = x.sin is one.
theorem fourier_sin_coeff_sin_one {
    fourier_sin_coeff(Real.sin, Nat.1) = Real.1
} by {
    forall(x: Real) {
        fourier_sin_integrand(Real.sin, Nat.1, x) = x.sin * sin_nt(Nat.1, x)
        sin_nt(Nat.1, x) = (from_nat[Real](Nat.1) * x).sin
        from_nat[Real](Nat.1) = Real.1
        from_nat[Real](Nat.1) * x = x
        sin_nt(Nat.1, x) = x.sin
        x.sin * sin_nt(Nat.1, x) = x.sin * x.sin
        fourier_sin_integrand(Real.sin, Nat.1, x) = x.sin * x.sin
        fn_sin_sq(x) = x.sin * x.sin
        fourier_sin_integrand(Real.sin, Nat.1, x) = fn_sin_sq(x)
    }
    function_extensionality(fourier_sin_integrand(Real.sin, Nat.1), fn_sin_sq)
    fourier_sin_integrand(Real.sin, Nat.1) = fn_sin_sq
    integral(fourier_sin_integrand(Real.sin, Nat.1), Real.0, two_pi) = integral(fn_sin_sq, Real.0, two_pi)
    integral_sin_sq_zero_two_pi
    integral(fn_sin_sq, Real.0, two_pi) = pi
    integral(fourier_sin_integrand(Real.sin, Nat.1), Real.0, two_pi) = pi
    fourier_sin_coeff(Real.sin, Nat.1) = integral(fourier_sin_integrand(Real.sin, Nat.1), Real.0, two_pi) / pi
    fourier_sin_coeff(Real.sin, Nat.1) = pi / pi
    pi_pos
    pi > Real.0
    lt_imp_ne(Real.0, pi)
    Real.0 < pi implies Real.0 != pi
    Real.0 != pi
    pi != Real.0
    div_mul_cancel_left(pi, Real.1)
    pi != Real.0 implies (pi * Real.1) / pi = Real.1
    (pi * Real.1) / pi = Real.1
    pi * Real.1 = pi
    pi / pi = Real.1
    fourier_sin_coeff(Real.sin, Nat.1) = Real.1
}

/// The b_2 coefficient of f(x) = x.sin is zero (orthogonality of Real.sin and /// (2x).sin).

theorem fourier_sin_coeff_sin_two {
    fourier_sin_coeff(Real.sin, Nat.2) = Real.0
} by {
    forall(x: Real) {
        fourier_sin_integrand(Real.sin, Nat.2, x) = x.sin * sin_nt(Nat.2, x)
        sin_nt(Nat.2, x) = (from_nat[Real](Nat.2) * x).sin
        from_nat_add[Real](Nat.1, Nat.1)
        from_nat[Real](Nat.1 + Nat.1) = from_nat[Real](Nat.1) + from_nat[Real](Nat.1)
        Nat.1 + Nat.1 = Nat.2
        from_nat[Real](Nat.2) = from_nat[Real](Nat.1) + from_nat[Real](Nat.1)
        from_nat[Real](Nat.1) = Real.1
        from_nat[Real](Nat.2) = Real.1 + Real.1
        Real.1 + Real.1 = two
        from_nat[Real](Nat.2) = two
        from_nat[Real](Nat.2) * x = two * x
        sin_nt(Nat.2, x) = (two * x).sin
        x.sin * sin_nt(Nat.2, x) = x.sin * (two * x).sin
        fourier_sin_integrand(Real.sin, Nat.2, x) = x.sin * (two * x).sin
        fn_sin_sin2(x) = x.sin * (two * x).sin
        fourier_sin_integrand(Real.sin, Nat.2, x) = fn_sin_sin2(x)
    }
    function_extensionality(fourier_sin_integrand(Real.sin, Nat.2), fn_sin_sin2)
    fourier_sin_integrand(Real.sin, Nat.2) = fn_sin_sin2
    integral(fourier_sin_integrand(Real.sin, Nat.2), Real.0, two_pi) = integral(fn_sin_sin2, Real.0, two_pi)
    integral_sin_sin2_zero
    integral(fn_sin_sin2, Real.0, two_pi) = Real.0
    integral(fourier_sin_integrand(Real.sin, Nat.2), Real.0, two_pi) = Real.0
    fourier_sin_coeff(Real.sin, Nat.2) = integral(fourier_sin_integrand(Real.sin, Nat.2), Real.0, two_pi) / pi
    fourier_sin_coeff(Real.sin, Nat.2) = Real.0 / pi
    Real.0 / pi = Real.0
    fourier_sin_coeff(Real.sin, Nat.2) = Real.0
}

/// The a_0 coefficient of f(x) = x.cos is zero.
theorem fourier_cos_coeff_cos_zero {
    fourier_cos_coeff(Real.cos, Nat.0) = Real.0
} by {
    forall(x: Real) {
        fourier_cos_integrand(Real.cos, Nat.0, x) = x.cos * cos_nt(Nat.0, x)
        cos_nt_zero_harmonic(x)
        cos_nt(Nat.0, x) = Real.1
        x.cos * cos_nt(Nat.0, x) = x.cos * Real.1
        x.cos * Real.1 = x.cos
        fourier_cos_integrand(Real.cos, Nat.0, x) = x.cos
    }
    function_extensionality(fourier_cos_integrand(Real.cos, Nat.0), Real.cos)
    fourier_cos_integrand(Real.cos, Nat.0) = Real.cos
    integral(fourier_cos_integrand(Real.cos, Nat.0), Real.0, two_pi) = integral(Real.cos, Real.0, two_pi)
    integral_cos_zero_period
    integral(Real.cos, Real.0, two_pi) = Real.0
    integral(fourier_cos_integrand(Real.cos, Nat.0), Real.0, two_pi) = Real.0
    fourier_cos_coeff(Real.cos, Nat.0) = integral(fourier_cos_integrand(Real.cos, Nat.0), Real.0, two_pi) / pi
    fourier_cos_coeff(Real.cos, Nat.0) = Real.0 / pi
    Real.0 / pi = Real.0
    fourier_cos_coeff(Real.cos, Nat.0) = Real.0
}

/// The a_1 coefficient of f(x) = x.cos is one.
theorem fourier_cos_coeff_cos_one {
    fourier_cos_coeff(Real.cos, Nat.1) = Real.1
} by {
    forall(x: Real) {
        fourier_cos_integrand(Real.cos, Nat.1, x) = x.cos * cos_nt(Nat.1, x)
        cos_nt(Nat.1, x) = (from_nat[Real](Nat.1) * x).cos
        from_nat[Real](Nat.1) = Real.1
        from_nat[Real](Nat.1) * x = x
        cos_nt(Nat.1, x) = x.cos
        x.cos * cos_nt(Nat.1, x) = x.cos * x.cos
        fourier_cos_integrand(Real.cos, Nat.1, x) = x.cos * x.cos
        fn_cos_sq(x) = x.cos * x.cos
        fourier_cos_integrand(Real.cos, Nat.1, x) = fn_cos_sq(x)
    }
    function_extensionality(fourier_cos_integrand(Real.cos, Nat.1), fn_cos_sq)
    fourier_cos_integrand(Real.cos, Nat.1) = fn_cos_sq
    integral(fourier_cos_integrand(Real.cos, Nat.1), Real.0, two_pi) = integral(fn_cos_sq, Real.0, two_pi)
    integral_cos_sq_zero_two_pi
    integral(fn_cos_sq, Real.0, two_pi) = pi
    integral(fourier_cos_integrand(Real.cos, Nat.1), Real.0, two_pi) = pi
    fourier_cos_coeff(Real.cos, Nat.1) = integral(fourier_cos_integrand(Real.cos, Nat.1), Real.0, two_pi) / pi
    fourier_cos_coeff(Real.cos, Nat.1) = pi / pi
    pi_pos
    pi > Real.0
    lt_imp_ne(Real.0, pi)
    Real.0 < pi implies Real.0 != pi
    Real.0 != pi
    pi != Real.0
    div_mul_cancel_left(pi, Real.1)
    pi != Real.0 implies (pi * Real.1) / pi = Real.1
    (pi * Real.1) / pi = Real.1
    pi * Real.1 = pi
    pi / pi = Real.1
    fourier_cos_coeff(Real.cos, Nat.1) = Real.1
}

/// The b_1 coefficient of f(x) = x.cos is zero.
theorem fourier_sin_coeff_cos_one {
    fourier_sin_coeff(Real.cos, Nat.1) = Real.0
} by {
    forall(x: Real) {
        fourier_sin_integrand(Real.cos, Nat.1, x) = x.cos * sin_nt(Nat.1, x)
        sin_nt(Nat.1, x) = (from_nat[Real](Nat.1) * x).sin
        from_nat[Real](Nat.1) = Real.1
        from_nat[Real](Nat.1) * x = x
        sin_nt(Nat.1, x) = x.sin
        x.cos * sin_nt(Nat.1, x) = x.cos * x.sin
        fourier_sin_integrand(Real.cos, Nat.1, x) = x.cos * x.sin
        fn_sincos(x) = x.sin * x.cos
        x.cos * x.sin = x.sin * x.cos
        fourier_sin_integrand(Real.cos, Nat.1, x) = fn_sincos(x)
    }
    function_extensionality(fourier_sin_integrand(Real.cos, Nat.1), fn_sincos)
    fourier_sin_integrand(Real.cos, Nat.1) = fn_sincos
    integral(fourier_sin_integrand(Real.cos, Nat.1), Real.0, two_pi) = integral(fn_sincos, Real.0, two_pi)
    integral_sin_mul_cos_zero
    integral(fn_sincos, Real.0, two_pi) = Real.0
    integral(fourier_sin_integrand(Real.cos, Nat.1), Real.0, two_pi) = Real.0
    fourier_sin_coeff(Real.cos, Nat.1) = integral(fourier_sin_integrand(Real.cos, Nat.1), Real.0, two_pi) / pi
    fourier_sin_coeff(Real.cos, Nat.1) = Real.0 / pi
    Real.0 / pi = Real.0
    fourier_sin_coeff(Real.cos, Nat.1) = Real.0
}

// ---------------------------------------------------------------------------
// The partial sums of the Fourier series
// ---------------------------------------------------------------------------

/// The n-th term of the Fourier partial sum of f:
/// a_n nx.cos + b_n nx.sin, for n >= 1 (the index is shifted).
define fourier_sum_term(f: Real -> Real, x: Real, n: Nat) -> Real {
    fourier_cos_coeff(f, n.suc) * cos_nt(n.suc, x) + fourier_sin_coeff(f, n.suc) * sin_nt(n.suc, x)
}

/// The N-th partial sum of the Fourier series of f:
/// S_N(x) = a_0 / 2 + sum_{n=1}^{N} (a_n nx.cos + b_n nx.sin).
define fourier_partial_sum(f: Real -> Real, n: Nat, x: Real) -> Real {
    fourier_cos_coeff(f, Nat.0) / two + partial(fourier_sum_term(f, x), n)
}

// ---------------------------------------------------------------------------
// Parseval's identity in the basic cases
// ---------------------------------------------------------------------------

/// pi / (2*pi) = 1/2.
theorem pi_div_two_pi {
    pi / two_pi = Real.one_half
} by {
    two_pi = pi + pi
    pi + pi = two * pi
    add_self_two(pi)
    two_pi = two * pi
    pi / two_pi = pi / (two * pi)
    half_twice(pi)
    (Real.1 / two) * (two * pi) = pi
    real_mul_comm(Real.1 / two, two * pi)
    (Real.1 / two) * (two * pi) = (two * pi) * (Real.1 / two)
    (two * pi) * (Real.1 / two) = pi
    two_pi_pos
    Real.0 < two_pi
    lt_imp_ne(Real.0, two_pi)
    Real.0 < two_pi implies Real.0 != two_pi
    Real.0 != two_pi
    two_pi != Real.0
    div_mul_cancel_left(two_pi, Real.one_half)
    two_pi != Real.0 implies (two_pi * Real.one_half) / two_pi = Real.one_half
    (two_pi * Real.one_half) / two_pi = Real.one_half
    Real.one_half = Real.1 / two
    (two * pi) * Real.one_half = (two * pi) * (Real.1 / two)
    real_mul_comm(two * pi, Real.1 / two)
    (two * pi) * (Real.1 / two) = (Real.1 / two) * (two * pi)
    (two * pi) * Real.one_half = (Real.1 / two) * (two * pi)
    half_twice(pi)
    (Real.1 / two) * (two * pi) = pi
    (two * pi) * Real.one_half = pi
    pi = (two * pi) * Real.one_half
    pi / (two * pi) = ((two * pi) * Real.one_half) / (two * pi)
    two_pi = two * pi
    (two * pi) * Real.one_half = two_pi * Real.one_half
    ((two * pi) * Real.one_half) / (two * pi) = (two_pi * Real.one_half) / two_pi
    pi / (two * pi) = (two_pi * Real.one_half) / two_pi
    (two_pi * Real.one_half) / two_pi = Real.one_half
    pi / (two * pi) = Real.one_half
    pi / two_pi = Real.one_half
}

/// Parseval for the sine harmonic: (1/2*pi) integral(Real.sin^2, 0, 2*pi) = 1/2.
theorem parseval_sin {
    mean_integral(fn_sin_sq) = Real.one_half
} by {
    mean_integral(fn_sin_sq) = integral(fn_sin_sq, Real.0, two_pi) / two_pi
    integral_sin_sq_zero_two_pi
    integral(fn_sin_sq, Real.0, two_pi) = pi
    mean_integral(fn_sin_sq) = pi / two_pi
    pi_div_two_pi
    pi / two_pi = Real.one_half
    mean_integral(fn_sin_sq) = Real.one_half
}

/// Parseval for the cosine harmonic: (1/2*pi) integral(Real.cos^2, 0, 2*pi) = 1/2.
theorem parseval_cos {
    mean_integral(fn_cos_sq) = Real.one_half
} by {
    mean_integral(fn_cos_sq) = integral(fn_cos_sq, Real.0, two_pi) / two_pi
    integral_cos_sq_zero_two_pi
    integral(fn_cos_sq, Real.0, two_pi) = pi
    mean_integral(fn_cos_sq) = pi / two_pi
    pi_div_two_pi
    pi / two_pi = Real.one_half
    mean_integral(fn_cos_sq) = Real.one_half
}

/// Parseval for the constant one: (1/2*pi) integral(1, 0, 2*pi) = 1.
theorem parseval_one {
    mean_integral(constant[Real, Real](Real.1)) = Real.1
} by {
    mean_integral(constant[Real, Real](Real.1)) = integral(constant[Real, Real](Real.1), Real.0, two_pi) / two_pi

    integral_one_zero_two_pi
    integral(constant[Real, Real](Real.1), Real.0, two_pi) = two_pi
    mean_integral(constant[Real, Real](Real.1)) = two_pi / two_pi
    two_pi_pos
    Real.0 < two_pi
    lt_imp_ne(Real.0, two_pi)
    Real.0 < two_pi implies Real.0 != two_pi
    Real.0 != two_pi
    two_pi != Real.0
    div_mul_cancel_left(two_pi, Real.1)
    two_pi != Real.0 implies (two_pi * Real.1) / two_pi = Real.1
    (two_pi * Real.1) / two_pi = Real.1
    two_pi * Real.1 = two_pi
    two_pi / two_pi = Real.1
    mean_integral(constant[Real, Real](Real.1)) = Real.1
}

// ---------------------------------------------------------------------------
// The norms of the general harmonics: integral(kx.sin^2) = pi and // integral(kx.cos^2) = pi for every k >= 1, with the cross term

// integral(kx.sin kx.cos) = 0.  These complete the orthogonality and // normalization of the trigonometric system for general k (the successor // formulation keeps every k >= 1 explicit).


// ---------------------------------------------------------------------------

/// The antiderivative of kx.sin^2: x -> x/2 - (2kx).sin/(4k), expressed with
/// the coefficient 1/(4k) = (1/2)/(2k).
define g_sin_nt_sq(k: Nat) -> (Real -> Real) {
    pointwise_add(pointwise_mul(constant[Real, Real](Real.one_half), identity_fn[Real]), pointwise_neg(pointwise_mul(constant[Real, Real](Real.one_half / (two * from_nat[Real](k))), sin_nt(Nat.2 * k))))


}

/// from_nat(2) = 2.
theorem from_nat_two {
    from_nat[Real](Nat.2) = two
} by {
    from_nat_add[Real](Nat.1, Nat.1)
    from_nat[Real](Nat.1 + Nat.1) = from_nat[Real](Nat.1) + from_nat[Real](Nat.1)
    Nat.1 + Nat.1 = Nat.2
    from_nat[Real](Nat.2) = from_nat[Real](Nat.1) + from_nat[Real](Nat.1)
    from_nat[Real](Nat.1) = Real.1
    from_nat[Real](Nat.2) = Real.1 + Real.1
    Real.1 + Real.1 = two
    from_nat[Real](Nat.2) = two
}

/// from_nat(2k) = 2 from_nat(k).
theorem from_nat_two_mul(k: Nat) {
    from_nat[Real](Nat.2 * k) = two * from_nat[Real](k)
} by {
    from_nat_mul[Real](Nat.2, k)
    from_nat[Real](Nat.2 * k) = from_nat[Real](Nat.2) * from_nat[Real](k)
    from_nat_two
    from_nat[Real](Nat.2) = two
    from_nat[Real](Nat.2 * k) = two * from_nat[Real](k)
}

/// (1/(4k)) * (2k) = 1/2 for k >= 1, in the coefficient form used by the
/// antiderivative of kx.sin^2.
theorem norm_coeff_times_two_k(k: Nat) {
    (Real.one_half / (two * from_nat[Real](k.suc))) * (two * from_nat[Real](k.suc)) = Real.one_half
} by {
    real_mul_comm(Real.one_half / (two * from_nat[Real](k.suc)), two * from_nat[Real](k.suc))
    (Real.one_half / (two * from_nat[Real](k.suc))) * (two * from_nat[Real](k.suc)) = (two * from_nat[Real](k.suc)) * (Real.one_half / (two * from_nat[Real](k.suc)))

    mul_div_left(two * from_nat[Real](k.suc), Real.one_half, two * from_nat[Real](k.suc))
    ((two * from_nat[Real](k.suc)) * Real.one_half) / (two * from_nat[Real](k.suc)) = (two * from_nat[Real](k.suc)) * (Real.one_half / (two * from_nat[Real](k.suc)))

    (two * from_nat[Real](k.suc)) * (Real.one_half / (two * from_nat[Real](k.suc))) = ((two * from_nat[Real](k.suc)) * Real.one_half) / (two * from_nat[Real](k.suc))


    from_nat_suc_pos_real(k)
    from_nat[Real](k.suc) > Real.0
    Real.0 < from_nat[Real](k.suc)
    lt_imp_ne(Real.0, from_nat[Real](k.suc))
    Real.0 < from_nat[Real](k.suc) implies Real.0 != from_nat[Real](k.suc)
    Real.0 != from_nat[Real](k.suc)
    from_nat[Real](k.suc) != Real.0
    two_positive
    two.is_positive
    pos_gt_zero(two)
    two > Real.0
    lt_imp_ne(Real.0, two)
    Real.0 < two implies Real.0 != two
    Real.0 != two
    two != Real.0
    mul_not_zero(two, from_nat[Real](k.suc))
    two != Real.0 and from_nat[Real](k.suc) != Real.0 implies two * from_nat[Real](k.suc) != Real.0
    two * from_nat[Real](k.suc) != Real.0
    div_mul_cancel_left(two * from_nat[Real](k.suc), Real.one_half)
    (two * from_nat[Real](k.suc)) != Real.0 implies ((two * from_nat[Real](k.suc)) * Real.one_half) / (two * from_nat[Real](k.suc)) = Real.one_half

    ((two * from_nat[Real](k.suc)) * Real.one_half) / (two * from_nat[Real](k.suc)) = Real.one_half
    (two * from_nat[Real](k.suc)) * (Real.one_half / (two * from_nat[Real](k.suc))) = Real.one_half
    (Real.one_half / (two * from_nat[Real](k.suc))) * (two * from_nat[Real](k.suc)) = Real.one_half
}

/// The half-angle identity for the harmonic: kx.sin^2 = (1 - (2kx).cos)/2.
theorem sin_nt_sq_half_angle(k: Nat, x: Real) {
    sin_nt(k.suc, x) * sin_nt(k.suc, x) = (Real.1 - cos_nt(Nat.2 * k.suc, x)) * Real.one_half
} by {
    sin_sq_times_two(from_nat[Real](k.suc) * x)
    two * ((from_nat[Real](k.suc) * x).sin * (from_nat[Real](k.suc) * x).sin) = Real.1 - (two * (from_nat[Real](k.suc) * x)).cos

    sin_nt(k.suc, x) = (from_nat[Real](k.suc) * x).sin
    (from_nat[Real](k.suc) * x).sin = sin_nt(k.suc, x)
    (from_nat[Real](k.suc) * x).sin * (from_nat[Real](k.suc) * x).sin = sin_nt(k.suc, x) * sin_nt(k.suc, x)

    two * ((from_nat[Real](k.suc) * x).sin * (from_nat[Real](k.suc) * x).sin) = two * (sin_nt(k.suc, x) * sin_nt(k.suc, x))

    two * (sin_nt(k.suc, x) * sin_nt(k.suc, x)) = Real.1 - (two * (from_nat[Real](k.suc) * x)).cos
    two * (from_nat[Real](k.suc) * x) = (two * from_nat[Real](k.suc)) * x
    mul_assoc(two, from_nat[Real](k.suc), x)
    two * (from_nat[Real](k.suc) * x) = (two * from_nat[Real](k.suc)) * x
    (two * (from_nat[Real](k.suc) * x)).cos = ((two * from_nat[Real](k.suc)) * x).cos
    two * (sin_nt(k.suc, x) * sin_nt(k.suc, x)) = Real.1 - ((two * from_nat[Real](k.suc)) * x).cos
    from_nat_two_mul(k.suc)
    from_nat[Real](Nat.2 * k.suc) = two * from_nat[Real](k.suc)
    two * from_nat[Real](k.suc) = from_nat[Real](Nat.2 * k.suc)
    (two * from_nat[Real](k.suc)) * x = from_nat[Real](Nat.2 * k.suc) * x
    ((two * from_nat[Real](k.suc)) * x).cos = (from_nat[Real](Nat.2 * k.suc) * x).cos
    cos_nt(Nat.2 * k.suc, x) = (from_nat[Real](Nat.2 * k.suc) * x).cos
    (from_nat[Real](Nat.2 * k.suc) * x).cos = cos_nt(Nat.2 * k.suc, x)
    two * (sin_nt(k.suc, x) * sin_nt(k.suc, x)) = Real.1 - cos_nt(Nat.2 * k.suc, x)
    mul_div_cancel(Real.1, two)
    two != Real.0 implies two * (Real.1 / two) = Real.1
    two != Real.0
    two * (Real.1 / two) = Real.1
    (Real.1 / two) * two = Real.1
    two_nonzero
    two != Real.0
    real_mul_comm(Real.one_half, two * (sin_nt(k.suc, x) * sin_nt(k.suc, x)))
    Real.one_half * (two * (sin_nt(k.suc, x) * sin_nt(k.suc, x))) = (two * (sin_nt(k.suc, x) * sin_nt(k.suc, x))) * Real.one_half

    Real.one_half = Real.1 / two
    (Real.1 / two) * (two * (sin_nt(k.suc, x) * sin_nt(k.suc, x))) = (two * (sin_nt(k.suc, x) * sin_nt(k.suc, x))) * Real.one_half

    half_twice(sin_nt(k.suc, x) * sin_nt(k.suc, x))
    (Real.1 / two) * (two * (sin_nt(k.suc, x) * sin_nt(k.suc, x))) = sin_nt(k.suc, x) * sin_nt(k.suc, x)
    sin_nt(k.suc, x) * sin_nt(k.suc, x) = (two * (sin_nt(k.suc, x) * sin_nt(k.suc, x))) * Real.one_half

    Real.one_half * (two * (sin_nt(k.suc, x) * sin_nt(k.suc, x))) = sin_nt(k.suc, x) * sin_nt(k.suc, x)

    (two * (sin_nt(k.suc, x) * sin_nt(k.suc, x))) * Real.one_half = (Real.1 - cos_nt(Nat.2 * k.suc, x)) * Real.one_half

    sin_nt(k.suc, x) * sin_nt(k.suc, x) = (Real.1 - cos_nt(Nat.2 * k.suc, x)) * Real.one_half
}

/// The derivative of the kx.sin^2 antiderivative is kx.sin^2.
theorem g_sin_nt_sq_derivative(k: Nat) {
    is_derivative_fn(g_sin_nt_sq(k.suc), pointwise_mul(sin_nt(k.suc), sin_nt(k.suc)))
} by {
    sin_nt_derivative(Nat.2 * k.suc)
    is_derivative_fn(sin_nt(Nat.2 * k.suc), pointwise_mul(constant[Real, Real](from_nat[Real](Nat.2 * k.suc)), cos_nt(Nat.2 * k.suc)))

    derivative_fn_const_mul(Real.one_half / (two * from_nat[Real](k.suc)), sin_nt(Nat.2 * k.suc), pointwise_mul(constant[Real, Real](from_nat[Real](Nat.2 * k.suc)), cos_nt(Nat.2 * k.suc)))


    is_derivative_fn(pointwise_mul(constant[Real, Real](Real.one_half / (two * from_nat[Real](k.suc))), sin_nt(Nat.2 * k.suc)), pointwise_mul(constant[Real, Real](Real.one_half / (two * from_nat[Real](k.suc))), pointwise_mul(constant[Real, Real](from_nat[Real](Nat.2 * k.suc)), cos_nt(Nat.2 * k.suc))))



    forall(x: Real) {
        pointwise_mul_apply(constant[Real, Real](Real.one_half / (two * from_nat[Real](k.suc))), pointwise_mul(constant[Real, Real](from_nat[Real](Nat.2 * k.suc)), cos_nt(Nat.2 * k.suc)), x)

        pointwise_mul(constant[Real, Real](Real.one_half / (two * from_nat[Real](k.suc))), pointwise_mul(constant[Real, Real](from_nat[Real](Nat.2 * k.suc)), cos_nt(Nat.2 * k.suc)), x) = constant[Real, Real](Real.one_half / (two * from_nat[Real](k.suc)), x) *


                pointwise_mul(constant[Real, Real](from_nat[Real](Nat.2 * k.suc)), cos_nt(Nat.2 * k.suc), x)
        constant[Real, Real](Real.one_half / (two * from_nat[Real](k.suc)), x) = Real.one_half / (two * from_nat[Real](k.suc))

        pointwise_mul_apply(constant[Real, Real](from_nat[Real](Nat.2 * k.suc)), cos_nt(Nat.2 * k.suc), x)
        pointwise_mul(constant[Real, Real](from_nat[Real](Nat.2 * k.suc)), cos_nt(Nat.2 * k.suc), x) = constant[Real, Real](from_nat[Real](Nat.2 * k.suc), x) * cos_nt(Nat.2 * k.suc, x)

        constant[Real, Real](from_nat[Real](Nat.2 * k.suc), x) = from_nat[Real](Nat.2 * k.suc)
        pointwise_mul(constant[Real, Real](from_nat[Real](Nat.2 * k.suc)), cos_nt(Nat.2 * k.suc), x) = from_nat[Real](Nat.2 * k.suc) * cos_nt(Nat.2 * k.suc, x)

        pointwise_mul(constant[Real, Real](Real.one_half / (two * from_nat[Real](k.suc))), pointwise_mul(constant[Real, Real](from_nat[Real](Nat.2 * k.suc)), cos_nt(Nat.2 * k.suc)), x) = (Real.one_half / (two * from_nat[Real](k.suc))) *


                (from_nat[Real](Nat.2 * k.suc) * cos_nt(Nat.2 * k.suc, x))
        from_nat_two_mul(k.suc)
        from_nat[Real](Nat.2 * k.suc) = two * from_nat[Real](k.suc)
        (Real.one_half / (two * from_nat[Real](k.suc))) * (two * from_nat[Real](k.suc)) = Real.one_half
        norm_coeff_times_two_k(k)
        mul_assoc(Real.one_half / (two * from_nat[Real](k.suc)), two * from_nat[Real](k.suc), cos_nt(Nat.2 * k.suc, x))

        (Real.one_half / (two * from_nat[Real](k.suc))) *
            ((two * from_nat[Real](k.suc)) * cos_nt(Nat.2 * k.suc, x)) = ((Real.one_half / (two * from_nat[Real](k.suc))) * (two * from_nat[Real](k.suc))) *

                cos_nt(Nat.2 * k.suc, x)
        ((Real.one_half / (two * from_nat[Real](k.suc))) * (two * from_nat[Real](k.suc))) *
            cos_nt(Nat.2 * k.suc, x) = Real.one_half * cos_nt(Nat.2 * k.suc, x)
        Real.one_half * cos_nt(Nat.2 * k.suc, x) = Real.one_half * cos_nt(Nat.2 * k.suc, x)
        (Real.one_half / (two * from_nat[Real](k.suc))) *
            (from_nat[Real](Nat.2 * k.suc) * cos_nt(Nat.2 * k.suc, x)) = Real.one_half * cos_nt(Nat.2 * k.suc, x)

        pointwise_mul(constant[Real, Real](Real.one_half / (two * from_nat[Real](k.suc))), pointwise_mul(constant[Real, Real](from_nat[Real](Nat.2 * k.suc)), cos_nt(Nat.2 * k.suc)), x) = Real.one_half * cos_nt(Nat.2 * k.suc, x)


        pointwise_mul_apply(constant[Real, Real](Real.one_half), cos_nt(Nat.2 * k.suc), x)
        pointwise_mul(constant[Real, Real](Real.one_half), cos_nt(Nat.2 * k.suc), x) = constant[Real, Real](Real.one_half, x) * cos_nt(Nat.2 * k.suc, x)

        constant[Real, Real](Real.one_half, x) = Real.one_half
        pointwise_mul(constant[Real, Real](Real.one_half), cos_nt(Nat.2 * k.suc), x) = Real.one_half * cos_nt(Nat.2 * k.suc, x)

        pointwise_mul(constant[Real, Real](Real.one_half / (two * from_nat[Real](k.suc))), pointwise_mul(constant[Real, Real](from_nat[Real](Nat.2 * k.suc)), cos_nt(Nat.2 * k.suc)), x) = pointwise_mul(constant[Real, Real](Real.one_half), cos_nt(Nat.2 * k.suc), x)


    }
    function_extensionality(pointwise_mul(constant[Real, Real](Real.one_half / (two * from_nat[Real](k.suc))), pointwise_mul(constant[Real, Real](from_nat[Real](Nat.2 * k.suc)), cos_nt(Nat.2 * k.suc))), pointwise_mul(constant[Real, Real](Real.one_half), cos_nt(Nat.2 * k.suc)))


    pointwise_mul(constant[Real, Real](Real.one_half / (two * from_nat[Real](k.suc))), pointwise_mul(constant[Real, Real](from_nat[Real](Nat.2 * k.suc)), cos_nt(Nat.2 * k.suc))) = pointwise_mul(constant[Real, Real](Real.one_half), cos_nt(Nat.2 * k.suc))


    is_derivative_fn(pointwise_mul(constant[Real, Real](Real.one_half / (two * from_nat[Real](k.suc))), sin_nt(Nat.2 * k.suc)), pointwise_mul(constant[Real, Real](Real.one_half), cos_nt(Nat.2 * k.suc)))


    derivative_fn_neg(pointwise_mul(constant[Real, Real](Real.one_half / (two * from_nat[Real](k.suc))), sin_nt(Nat.2 * k.suc)), pointwise_mul(constant[Real, Real](Real.one_half), cos_nt(Nat.2 * k.suc)))


    is_derivative_fn(pointwise_neg(pointwise_mul(constant[Real, Real](Real.one_half / (two * from_nat[Real](k.suc))), sin_nt(Nat.2 * k.suc))), pointwise_neg(pointwise_mul(constant[Real, Real](Real.one_half), cos_nt(Nat.2 * k.suc))))


    derivative_fn_const_mul(Real.one_half, identity_fn[Real], constant[Real, Real](Real.1))
    is_derivative_fn(pointwise_mul(constant[Real, Real](Real.one_half), identity_fn[Real]), pointwise_mul(constant[Real, Real](Real.one_half), constant[Real, Real](Real.1)))

    forall(x: Real) {
        pointwise_mul_apply(constant[Real, Real](Real.one_half), constant[Real, Real](Real.1), x)
        pointwise_mul(constant[Real, Real](Real.one_half), constant[Real, Real](Real.1), x) = constant[Real, Real](Real.one_half, x) * constant[Real, Real](Real.1, x)

        constant[Real, Real](Real.one_half, x) = Real.one_half
        constant[Real, Real](Real.1, x) = Real.1
        pointwise_mul(constant[Real, Real](Real.one_half), constant[Real, Real](Real.1), x) = Real.one_half * Real.1

        Real.one_half * Real.1 = Real.one_half
        pointwise_mul(constant[Real, Real](Real.one_half), constant[Real, Real](Real.1), x) = Real.one_half
        constant[Real, Real](Real.one_half, x) = Real.one_half
        pointwise_mul(constant[Real, Real](Real.one_half), constant[Real, Real](Real.1), x) = constant[Real, Real](Real.one_half, x)

    }
    function_extensionality(pointwise_mul(constant[Real, Real](Real.one_half), constant[Real, Real](Real.1)), constant[Real, Real](Real.one_half))

    pointwise_mul(constant[Real, Real](Real.one_half), constant[Real, Real](Real.1)) = constant[Real, Real](Real.one_half)

    is_derivative_fn(pointwise_mul(constant[Real, Real](Real.one_half), identity_fn[Real]), constant[Real, Real](Real.one_half))

    derivative_fn_add(pointwise_mul(constant[Real, Real](Real.one_half), identity_fn[Real]), pointwise_neg(pointwise_mul(constant[Real, Real](Real.one_half / (two * from_nat[Real](k.suc))), sin_nt(Nat.2 * k.suc))), constant[Real, Real](Real.one_half), pointwise_neg(pointwise_mul(constant[Real, Real](Real.one_half), cos_nt(Nat.2 * k.suc))))




    is_derivative_fn(g_sin_nt_sq(k.suc), pointwise_add(constant[Real, Real](Real.one_half), pointwise_neg(pointwise_mul(constant[Real, Real](Real.one_half), cos_nt(Nat.2 * k.suc)))))


    forall(x: Real) {
        pointwise_add_apply(constant[Real, Real](Real.one_half), pointwise_neg(pointwise_mul(constant[Real, Real](Real.one_half), cos_nt(Nat.2 * k.suc))), x)

        pointwise_add(constant[Real, Real](Real.one_half), pointwise_neg(pointwise_mul(constant[Real, Real](Real.one_half), cos_nt(Nat.2 * k.suc))), x) = constant[Real, Real](Real.one_half, x) +


                pointwise_neg(pointwise_mul(constant[Real, Real](Real.one_half), cos_nt(Nat.2 * k.suc)), x)
        constant[Real, Real](Real.one_half, x) = Real.one_half
        pointwise_neg_apply(pointwise_mul(constant[Real, Real](Real.one_half), cos_nt(Nat.2 * k.suc)), x)
        pointwise_neg(pointwise_mul(constant[Real, Real](Real.one_half), cos_nt(Nat.2 * k.suc)), x) = -(pointwise_mul(constant[Real, Real](Real.one_half), cos_nt(Nat.2 * k.suc), x))

        pointwise_mul_apply(constant[Real, Real](Real.one_half), cos_nt(Nat.2 * k.suc), x)
        pointwise_mul(constant[Real, Real](Real.one_half), cos_nt(Nat.2 * k.suc), x) = constant[Real, Real](Real.one_half, x) * cos_nt(Nat.2 * k.suc, x)

        constant[Real, Real](Real.one_half, x) = Real.one_half
        pointwise_mul(constant[Real, Real](Real.one_half), cos_nt(Nat.2 * k.suc), x) = Real.one_half * cos_nt(Nat.2 * k.suc, x)

        pointwise_neg(pointwise_mul(constant[Real, Real](Real.one_half), cos_nt(Nat.2 * k.suc)), x) = -(Real.one_half * cos_nt(Nat.2 * k.suc, x))

        pointwise_add(constant[Real, Real](Real.one_half), pointwise_neg(pointwise_mul(constant[Real, Real](Real.one_half), cos_nt(Nat.2 * k.suc))), x) = Real.one_half + -(Real.one_half * cos_nt(Nat.2 * k.suc, x))

        Real.one_half * Real.1 = Real.one_half
        Real.one_half + -(Real.one_half * cos_nt(Nat.2 * k.suc, x)) = Real.one_half * Real.1 + -(Real.one_half * cos_nt(Nat.2 * k.suc, x))

        mul_neg_right(Real.one_half, cos_nt(Nat.2 * k.suc, x))
        Real.one_half * (-cos_nt(Nat.2 * k.suc, x)) = -(Real.one_half * cos_nt(Nat.2 * k.suc, x))
        Real.one_half * Real.1 + -(Real.one_half * cos_nt(Nat.2 * k.suc, x)) = Real.one_half * Real.1 + Real.one_half * (-cos_nt(Nat.2 * k.suc, x))

        mul_distrib_left(Real.one_half, Real.1, -cos_nt(Nat.2 * k.suc, x))
        Real.one_half * (Real.1 + -cos_nt(Nat.2 * k.suc, x)) = Real.one_half * Real.1 + Real.one_half * (-cos_nt(Nat.2 * k.suc, x))

        Real.one_half * Real.1 + Real.one_half * (-cos_nt(Nat.2 * k.suc, x)) = Real.one_half * (Real.1 + -cos_nt(Nat.2 * k.suc, x))

        Real.1 + -cos_nt(Nat.2 * k.suc, x) = Real.1 - cos_nt(Nat.2 * k.suc, x)
        Real.one_half * (Real.1 + -cos_nt(Nat.2 * k.suc, x)) = Real.one_half * (Real.1 - cos_nt(Nat.2 * k.suc, x))

        Real.one_half + -(Real.one_half * cos_nt(Nat.2 * k.suc, x)) = Real.one_half * (Real.1 - cos_nt(Nat.2 * k.suc, x))

        real_mul_comm(Real.one_half, Real.1 - cos_nt(Nat.2 * k.suc, x))
        Real.one_half * (Real.1 - cos_nt(Nat.2 * k.suc, x)) = (Real.1 - cos_nt(Nat.2 * k.suc, x)) * Real.one_half

        Real.one_half + -(Real.one_half * cos_nt(Nat.2 * k.suc, x)) = (Real.1 - cos_nt(Nat.2 * k.suc, x)) * Real.one_half

        pointwise_add(constant[Real, Real](Real.one_half), pointwise_neg(pointwise_mul(constant[Real, Real](Real.one_half), cos_nt(Nat.2 * k.suc))), x) = (Real.1 - cos_nt(Nat.2 * k.suc, x)) * Real.one_half


        sin_nt_sq_half_angle(k, x)
        sin_nt(k.suc, x) * sin_nt(k.suc, x) = (Real.1 - cos_nt(Nat.2 * k.suc, x)) * Real.one_half
        pointwise_add(constant[Real, Real](Real.one_half), pointwise_neg(pointwise_mul(constant[Real, Real](Real.one_half), cos_nt(Nat.2 * k.suc))), x) = sin_nt(k.suc, x) * sin_nt(k.suc, x)


        pointwise_mul_apply(sin_nt(k.suc), sin_nt(k.suc), x)
        pointwise_mul(sin_nt(k.suc), sin_nt(k.suc), x) = sin_nt(k.suc, x) * sin_nt(k.suc, x)
        pointwise_add(constant[Real, Real](Real.one_half), pointwise_neg(pointwise_mul(constant[Real, Real](Real.one_half), cos_nt(Nat.2 * k.suc))), x) = pointwise_mul(sin_nt(k.suc), sin_nt(k.suc), x)


    }
    function_extensionality(pointwise_add(constant[Real, Real](Real.one_half), pointwise_neg(pointwise_mul(constant[Real, Real](Real.one_half), cos_nt(Nat.2 * k.suc)))), pointwise_mul(sin_nt(k.suc), sin_nt(k.suc)))


    pointwise_add(constant[Real, Real](Real.one_half), pointwise_neg(pointwise_mul(constant[Real, Real](Real.one_half), cos_nt(Nat.2 * k.suc)))) = pointwise_mul(sin_nt(k.suc), sin_nt(k.suc))


    is_derivative_fn(g_sin_nt_sq(k.suc), pointwise_mul(sin_nt(k.suc), sin_nt(k.suc)))
}

/// The antiderivative g_sin_nt_sq(k.suc) is continuous.
theorem g_sin_nt_sq_continuous(k: Nat) {
    continuous(g_sin_nt_sq(k.suc))
} by {
    constant_function_is_continuous(Real.one_half)
    continuous(constant[Real, Real](Real.one_half))
    identity_function_is_continuous
    continuous(identity_fn[Real])
    continuous_pointwise_mul(constant[Real, Real](Real.one_half), identity_fn[Real])
    continuous(constant[Real, Real](Real.one_half)) and continuous(identity_fn[Real]) implies continuous(pointwise_mul(constant[Real, Real](Real.one_half), identity_fn[Real]))

    continuous(pointwise_mul(constant[Real, Real](Real.one_half), identity_fn[Real]))
    constant_function_is_continuous(Real.one_half / (two * from_nat[Real](k.suc)))
    continuous(constant[Real, Real](Real.one_half / (two * from_nat[Real](k.suc))))
    sin_nt_continuous(Nat.2 * k.suc)
    continuous(sin_nt(Nat.2 * k.suc))
    continuous_pointwise_mul(constant[Real, Real](Real.one_half / (two * from_nat[Real](k.suc))), sin_nt(Nat.2 * k.suc))

    continuous(constant[Real, Real](Real.one_half / (two * from_nat[Real](k.suc)))) and continuous(sin_nt(Nat.2 * k.suc)) implies continuous(pointwise_mul(constant[Real, Real](Real.one_half / (two * from_nat[Real](k.suc))), sin_nt(Nat.2 * k.suc)))
    continuous(pointwise_mul(constant[Real, Real](Real.one_half / (two * from_nat[Real](k.suc))), sin_nt(Nat.2 * k.suc)))

    continuous_pointwise_neg(pointwise_mul(constant[Real, Real](Real.one_half / (two * from_nat[Real](k.suc))), sin_nt(Nat.2 * k.suc)))

    continuous(pointwise_neg(pointwise_mul(constant[Real, Real](Real.one_half / (two * from_nat[Real](k.suc))), sin_nt(Nat.2 * k.suc))))

    continuous_pointwise_add(pointwise_mul(constant[Real, Real](Real.one_half), identity_fn[Real]), pointwise_neg(pointwise_mul(constant[Real, Real](Real.one_half / (two * from_nat[Real](k.suc))), sin_nt(Nat.2 * k.suc))))


    continuous(pointwise_mul(constant[Real, Real](Real.one_half), identity_fn[Real])) and continuous(pointwise_neg(pointwise_mul(constant[Real, Real](Real.one_half / (two * from_nat[Real](k.suc))),  sin_nt(Nat.2 * k.suc)))) implies continuous(pointwise_add(pointwise_mul(constant[Real, Real](Real.one_half), identity_fn[Real]), pointwise_neg(pointwise_mul(constant[Real, Real](Real.one_half / (two * from_nat[Real](k.suc))), sin_nt(Nat.2 * k.suc)))))



    continuous(pointwise_add(pointwise_mul(constant[Real, Real](Real.one_half), identity_fn[Real]), pointwise_neg(pointwise_mul(constant[Real, Real](Real.one_half / (two * from_nat[Real](k.suc))), sin_nt(Nat.2 * k.suc)))))


    g_sin_nt_sq(k.suc) = pointwise_add(pointwise_mul(constant[Real, Real](Real.one_half), identity_fn[Real]), pointwise_neg(pointwise_mul(constant[Real, Real](Real.one_half / (two * from_nat[Real](k.suc))), sin_nt(Nat.2 * k.suc))))


    continuous(g_sin_nt_sq(k.suc))
}

/// The square of the sine harmonic is bounded by one in absolute value.
theorem sin_nt_sq_abs_le_one(k: Nat, x: Real) {
    (pointwise_mul(sin_nt(k.suc), sin_nt(k.suc), x)).abs <= Real.1
} by {
    pointwise_mul_apply(sin_nt(k.suc), sin_nt(k.suc), x)
    pointwise_mul(sin_nt(k.suc), sin_nt(k.suc), x) = sin_nt(k.suc, x) * sin_nt(k.suc, x)
    (pointwise_mul(sin_nt(k.suc), sin_nt(k.suc), x)).abs = (sin_nt(k.suc, x) * sin_nt(k.suc, x)).abs
    sin_nt_abs_le_one(k.suc, x)
    (sin_nt(k.suc, x)).abs <= Real.1
    abs_mul_le_one(sin_nt(k.suc, x), sin_nt(k.suc, x))
    (sin_nt(k.suc, x)).abs <= Real.1 and (sin_nt(k.suc, x)).abs <= Real.1 implies (sin_nt(k.suc, x) * sin_nt(k.suc, x)).abs <= Real.1

    (sin_nt(k.suc, x) * sin_nt(k.suc, x)).abs <= Real.1
    (pointwise_mul(sin_nt(k.suc), sin_nt(k.suc), x)).abs <= Real.1
}

/// The square of the sine harmonic is 2k-Lipschitz.
theorem sin_nt_sq_lipschitz(k: Nat, u: Real, v: Real) {
    (pointwise_mul(sin_nt(k.suc), sin_nt(k.suc), u) - pointwise_mul(sin_nt(k.suc), sin_nt(k.suc), v)).abs <= (two * from_nat[Real](k.suc)) * (u - v).abs

} by {
    pointwise_mul_apply(sin_nt(k.suc), sin_nt(k.suc), u)
    pointwise_mul(sin_nt(k.suc), sin_nt(k.suc), u) = sin_nt(k.suc, u) * sin_nt(k.suc, u)
    pointwise_mul_apply(sin_nt(k.suc), sin_nt(k.suc), v)
    pointwise_mul(sin_nt(k.suc), sin_nt(k.suc), v) = sin_nt(k.suc, v) * sin_nt(k.suc, v)
    pointwise_mul(sin_nt(k.suc), sin_nt(k.suc), u) - pointwise_mul(sin_nt(k.suc), sin_nt(k.suc), v) = sin_nt(k.suc, u) * sin_nt(k.suc, u) - sin_nt(k.suc, v) * sin_nt(k.suc, v)

    mul_sub_distrib_left(sin_nt(k.suc, u), sin_nt(k.suc, v), sin_nt(k.suc, u) + sin_nt(k.suc, v))
    (sin_nt(k.suc, u) - sin_nt(k.suc, v)) * (sin_nt(k.suc, u) + sin_nt(k.suc, v)) = sin_nt(k.suc, u) * (sin_nt(k.suc, u) + sin_nt(k.suc, v)) -

            sin_nt(k.suc, v) * (sin_nt(k.suc, u) + sin_nt(k.suc, v))
    mul_distrib_right(sin_nt(k.suc, u), sin_nt(k.suc, u), sin_nt(k.suc, v))
    sin_nt(k.suc, u) * (sin_nt(k.suc, u) + sin_nt(k.suc, v)) = sin_nt(k.suc, u) * sin_nt(k.suc, u) + sin_nt(k.suc, u) * sin_nt(k.suc, v)

    mul_distrib_right(sin_nt(k.suc, v), sin_nt(k.suc, u), sin_nt(k.suc, v))
    sin_nt(k.suc, v) * (sin_nt(k.suc, u) + sin_nt(k.suc, v)) = sin_nt(k.suc, v) * sin_nt(k.suc, u) + sin_nt(k.suc, v) * sin_nt(k.suc, v)

    (sin_nt(k.suc, u) - sin_nt(k.suc, v)) * (sin_nt(k.suc, u) + sin_nt(k.suc, v)) = (sin_nt(k.suc, u) * sin_nt(k.suc, u) + sin_nt(k.suc, u) * sin_nt(k.suc, v)) -

            (sin_nt(k.suc, v) * sin_nt(k.suc, u) + sin_nt(k.suc, v) * sin_nt(k.suc, v))
    sin_nt(k.suc, u) * sin_nt(k.suc, v) = sin_nt(k.suc, v) * sin_nt(k.suc, u)
    (sin_nt(k.suc, u) * sin_nt(k.suc, u) + sin_nt(k.suc, v) * sin_nt(k.suc, u)) -
        (sin_nt(k.suc, v) * sin_nt(k.suc, u) + sin_nt(k.suc, v) * sin_nt(k.suc, v)) = sin_nt(k.suc, u) * sin_nt(k.suc, u) - sin_nt(k.suc, v) * sin_nt(k.suc, v)

    add_sub_add_cancel(sin_nt(k.suc, u) * sin_nt(k.suc, u), sin_nt(k.suc, v) * sin_nt(k.suc, u), sin_nt(k.suc, v) * sin_nt(k.suc, v))

    (sin_nt(k.suc, u) * sin_nt(k.suc, u) + sin_nt(k.suc, v) * sin_nt(k.suc, u)) -
        (sin_nt(k.suc, v) * sin_nt(k.suc, u) + sin_nt(k.suc, v) * sin_nt(k.suc, v)) = sin_nt(k.suc, u) * sin_nt(k.suc, u) - sin_nt(k.suc, v) * sin_nt(k.suc, v)

    (sin_nt(k.suc, u) - sin_nt(k.suc, v)) * (sin_nt(k.suc, u) + sin_nt(k.suc, v)) = sin_nt(k.suc, u) * sin_nt(k.suc, u) - sin_nt(k.suc, v) * sin_nt(k.suc, v)

    (pointwise_mul(sin_nt(k.suc), sin_nt(k.suc), u) - pointwise_mul(sin_nt(k.suc), sin_nt(k.suc), v)).abs <= ((sin_nt(k.suc, u) - sin_nt(k.suc, v)) * (sin_nt(k.suc, u) + sin_nt(k.suc, v))).abs

    mul_abs(sin_nt(k.suc, u) - sin_nt(k.suc, v), sin_nt(k.suc, u) + sin_nt(k.suc, v))
    ((sin_nt(k.suc, u) - sin_nt(k.suc, v)) * (sin_nt(k.suc, u) + sin_nt(k.suc, v))).abs = (sin_nt(k.suc, u) - sin_nt(k.suc, v)).abs * (sin_nt(k.suc, u) + sin_nt(k.suc, v)).abs

    (pointwise_mul(sin_nt(k.suc), sin_nt(k.suc), u) - pointwise_mul(sin_nt(k.suc), sin_nt(k.suc), v)).abs <= (sin_nt(k.suc, u) - sin_nt(k.suc, v)).abs * (sin_nt(k.suc, u) + sin_nt(k.suc, v)).abs

    sin_nt_lipschitz(k.suc, u, v)
    (sin_nt(k.suc, u) - sin_nt(k.suc, v)).abs <= from_nat[Real](k.suc) * (u - v).abs
    abs_gte_zero(sin_nt(k.suc, u) + sin_nt(k.suc, v))
    Real.0 <= (sin_nt(k.suc, u) + sin_nt(k.suc, v)).abs
    mul_le_mul_of_nonneg_right((sin_nt(k.suc, u) - sin_nt(k.suc, v)).abs, from_nat[Real](k.suc) * (u - v).abs, (sin_nt(k.suc, u) + sin_nt(k.suc, v)).abs)

    (sin_nt(k.suc, u) - sin_nt(k.suc, v)).abs * (sin_nt(k.suc, u) + sin_nt(k.suc, v)).abs <= (from_nat[Real](k.suc) * (u - v).abs) * (sin_nt(k.suc, u) + sin_nt(k.suc, v)).abs

    sin_add_abs_le_two(from_nat[Real](k.suc) * u, from_nat[Real](k.suc) * v)
    ((from_nat[Real](k.suc) * u).sin + (from_nat[Real](k.suc) * v).sin).abs <= two
    sin_nt(k.suc, u) = (from_nat[Real](k.suc) * u).sin
    (from_nat[Real](k.suc) * u).sin = sin_nt(k.suc, u)
    sin_nt(k.suc, v) = (from_nat[Real](k.suc) * v).sin
    (from_nat[Real](k.suc) * v).sin = sin_nt(k.suc, v)
    ((from_nat[Real](k.suc) * u).sin + (from_nat[Real](k.suc) * v).sin).abs = (sin_nt(k.suc, u) + sin_nt(k.suc, v)).abs

    (sin_nt(k.suc, u) + sin_nt(k.suc, v)).abs <= two
    from_nat_nonneg_real(k.suc)
    Real.0 <= from_nat[Real](k.suc)
    abs_gte_zero((u - v).abs)
    Real.0 <= (u - v).abs
    from_nat_nonneg_real(k.suc)
    Real.0 <= from_nat[Real](k.suc)
    from_nat[Real](k.suc) >= Real.0
    (u - v).abs >= Real.0
    mul_nonneg(from_nat[Real](k.suc), (u - v).abs)
    from_nat[Real](k.suc) >= Real.0 and (u - v).abs >= Real.0 implies from_nat[Real](k.suc) * (u - v).abs >= Real.0
    from_nat[Real](k.suc) * (u - v).abs >= Real.0
    Real.0 <= from_nat[Real](k.suc) * (u - v).abs
    mul_le_mul_of_nonneg_left((sin_nt(k.suc, u) + sin_nt(k.suc, v)).abs, two, from_nat[Real](k.suc) * (u - v).abs)

    (from_nat[Real](k.suc) * (u - v).abs) * (sin_nt(k.suc, u) + sin_nt(k.suc, v)).abs <= (from_nat[Real](k.suc) * (u - v).abs) * two
    lte_trans[Real]((sin_nt(k.suc, u) - sin_nt(k.suc, v)).abs * (sin_nt(k.suc, u) + sin_nt(k.suc, v)).abs, (from_nat[Real](k.suc) * (u - v).abs) * (sin_nt(k.suc, u) + sin_nt(k.suc, v)).abs, (from_nat[Real](k.suc) * (u - v).abs) * two)


    (sin_nt(k.suc, u) - sin_nt(k.suc, v)).abs * (sin_nt(k.suc, u) + sin_nt(k.suc, v)).abs <= (from_nat[Real](k.suc) * (u - v).abs) * two

    (from_nat[Real](k.suc) * (u - v).abs) * two = two * (from_nat[Real](k.suc) * (u - v).abs)
    (sin_nt(k.suc, u) - sin_nt(k.suc, v)).abs * (sin_nt(k.suc, u) + sin_nt(k.suc, v)).abs <= two * (from_nat[Real](k.suc) * (u - v).abs)

    mul_assoc(two, from_nat[Real](k.suc), (u - v).abs)
    two * (from_nat[Real](k.suc) * (u - v).abs) = (two * from_nat[Real](k.suc)) * (u - v).abs
    (sin_nt(k.suc, u) - sin_nt(k.suc, v)).abs * (sin_nt(k.suc, u) + sin_nt(k.suc, v)).abs <= (two * from_nat[Real](k.suc)) * (u - v).abs

    lte_trans[Real]((pointwise_mul(sin_nt(k.suc), sin_nt(k.suc), u) - pointwise_mul(sin_nt(k.suc), sin_nt(k.suc), v)).abs, (sin_nt(k.suc, u) - sin_nt(k.suc, v)).abs * (sin_nt(k.suc, u) + sin_nt(k.suc, v)).abs, (two * from_nat[Real](k.suc)) * (u - v).abs)


    (pointwise_mul(sin_nt(k.suc), sin_nt(k.suc), u) - pointwise_mul(sin_nt(k.suc), sin_nt(k.suc), v)).abs <= (two * from_nat[Real](k.suc)) * (u - v).abs

}

/// The square of the sine harmonic is integrable on [0, 2*pi].
theorem sin_nt_sq_integrable(k: Nat) {
    is_integrable(pointwise_mul(sin_nt(k.suc), sin_nt(k.suc)), Real.0, two_pi)
} by {
        two_pi_nonneg
        Real.0 <= two_pi
        from_nat_nonneg_real(k.suc)
        Real.0 <= from_nat[Real](k.suc)
        two_positive
        two.is_positive
        pos_gt_zero(two)
        two > Real.0
        lt_imp_lte(Real.0, two)
        Real.0 <= two
        two >= Real.0
        from_nat[Real](k.suc) >= Real.0
        mul_nonneg(two, from_nat[Real](k.suc))
        two >= Real.0 and from_nat[Real](k.suc) >= Real.0 implies two * from_nat[Real](k.suc) >= Real.0
        two * from_nat[Real](k.suc) >= Real.0
        Real.0 <= two * from_nat[Real](k.suc)
        g_sin_nt_sq_continuous(k)
        continuous(g_sin_nt_sq(k.suc))
        g_sin_nt_sq_derivative(k)
        is_derivative_fn(g_sin_nt_sq(k.suc), pointwise_mul(sin_nt(k.suc), sin_nt(k.suc)))
        forall(u: Real, v: Real) {
            if interval_contains(Real.0, two_pi, u) and interval_contains(Real.0, two_pi, v) {
                sin_nt_sq_lipschitz(k, u, v)
                (pointwise_mul(sin_nt(k.suc), sin_nt(k.suc), u) - pointwise_mul(sin_nt(k.suc), sin_nt(k.suc), v)).abs <= (two * from_nat[Real](k.suc)) * (u - v).abs

            }
        }
        forall(t: Real) {
            if interval_contains(Real.0, two_pi, t) {
                sin_nt_sq_abs_le_one(k, t)
                (pointwise_mul(sin_nt(k.suc), sin_nt(k.suc), t)).abs <= Real.1
                abs_le_one_imp_lower(pointwise_mul(sin_nt(k.suc), sin_nt(k.suc), t))
                -Real.1 <= pointwise_mul(sin_nt(k.suc), sin_nt(k.suc), t)
            }
        }
        forall(t: Real) {
            if interval_contains(Real.0, two_pi, t) {
                sin_nt_sq_abs_le_one(k, t)
                (pointwise_mul(sin_nt(k.suc), sin_nt(k.suc), t)).abs <= Real.1
                abs_le_one_imp_upper(pointwise_mul(sin_nt(k.suc), sin_nt(k.suc), t))
                pointwise_mul(sin_nt(k.suc), sin_nt(k.suc), t) <= Real.1
            }
        }
        Real.0 <= two_pi
        (Real.0 <= two_pi) and Real.0 <= two * from_nat[Real](k.suc)
        ((Real.0 <= two_pi) and Real.0 <= two * from_nat[Real](k.suc)) and continuous(g_sin_nt_sq(k.suc))
        (((Real.0 <= two_pi) and Real.0 <= two * from_nat[Real](k.suc)) and continuous(g_sin_nt_sq(k.suc))) and is_derivative_fn(g_sin_nt_sq(k.suc), pointwise_mul(sin_nt(k.suc), sin_nt(k.suc)))

        ((((Real.0 <= two_pi) and Real.0 <= two * from_nat[Real](k.suc)) and continuous(g_sin_nt_sq(k.suc))) and is_derivative_fn(g_sin_nt_sq(k.suc), pointwise_mul(sin_nt(k.suc), sin_nt(k.suc)))) and forall(u: Real, v: Real) {


                interval_contains(Real.0, two_pi, u) and interval_contains(Real.0, two_pi, v)
                implies (pointwise_mul(sin_nt(k.suc), sin_nt(k.suc), u) - pointwise_mul(sin_nt(k.suc), sin_nt(k.suc), v)).abs <= (two * from_nat[Real](k.suc)) * (u - v).abs

            }
        (((((Real.0 <= two_pi) and Real.0 <= two * from_nat[Real](k.suc)) and continuous(g_sin_nt_sq(k.suc))) and is_derivative_fn(g_sin_nt_sq(k.suc), pointwise_mul(sin_nt(k.suc), sin_nt(k.suc)))) and forall(u: Real, v: Real) {


                interval_contains(Real.0, two_pi, u) and interval_contains(Real.0, two_pi, v)
                implies (pointwise_mul(sin_nt(k.suc), sin_nt(k.suc), u) - pointwise_mul(sin_nt(k.suc), sin_nt(k.suc), v)).abs <= (two * from_nat[Real](k.suc)) * (u - v).abs

            }) and forall(t: Real) {
                interval_contains(Real.0, two_pi, t) implies -Real.1 <= pointwise_mul(sin_nt(k.suc), sin_nt(k.suc), t)
            }
        ((((((Real.0 <= two_pi) and Real.0 <= two * from_nat[Real](k.suc)) and continuous(g_sin_nt_sq(k.suc))) and is_derivative_fn(g_sin_nt_sq(k.suc), pointwise_mul(sin_nt(k.suc), sin_nt(k.suc)))) and forall(u: Real, v: Real) {


                interval_contains(Real.0, two_pi, u) and interval_contains(Real.0, two_pi, v)
                implies (pointwise_mul(sin_nt(k.suc), sin_nt(k.suc), u) - pointwise_mul(sin_nt(k.suc), sin_nt(k.suc), v)).abs <= (two * from_nat[Real](k.suc)) * (u - v).abs

            }) and forall(t: Real) {
                interval_contains(Real.0, two_pi, t) implies -Real.1 <= pointwise_mul(sin_nt(k.suc), sin_nt(k.suc), t)
            }) and forall(t: Real) {
                interval_contains(Real.0, two_pi, t) implies pointwise_mul(sin_nt(k.suc), sin_nt(k.suc), t) <= Real.1
            }
        if ((((((Real.0 <= two_pi) and Real.0 <= two * from_nat[Real](k.suc)) and continuous(g_sin_nt_sq(k.suc))) and is_derivative_fn(g_sin_nt_sq(k.suc), pointwise_mul(sin_nt(k.suc), sin_nt(k.suc)))) and forall(u: Real, v: Real) {


                interval_contains(Real.0, two_pi, u) and interval_contains(Real.0, two_pi, v)
                implies (pointwise_mul(sin_nt(k.suc), sin_nt(k.suc), u) - pointwise_mul(sin_nt(k.suc), sin_nt(k.suc), v)).abs <= (two * from_nat[Real](k.suc)) * (u - v).abs

            }) and forall(t: Real) {
                interval_contains(Real.0, two_pi, t) implies -Real.1 <= pointwise_mul(sin_nt(k.suc), sin_nt(k.suc), t)
            }) and forall(t: Real) {
                interval_contains(Real.0, two_pi, t) implies pointwise_mul(sin_nt(k.suc), sin_nt(k.suc), t) <= Real.1
            } {
            fn_integrable_gen(pointwise_mul(sin_nt(k.suc), sin_nt(k.suc)), g_sin_nt_sq(k.suc), Real.0, two_pi, two * from_nat[Real](k.suc), -Real.1, Real.1)

            is_integrable(pointwise_mul(sin_nt(k.suc), sin_nt(k.suc)), Real.0, two_pi)
        }
}

/// The integral of kx.sin^2 over [0, 2*pi] is pi, for k >= 1.
theorem integral_sin_nt_sq(k: Nat) {
    integral(pointwise_mul(sin_nt(k.suc), sin_nt(k.suc)), Real.0, two_pi) = pi
} by {
        two_pi_nonneg
        Real.0 <= two_pi
        g_sin_nt_sq_continuous(k)
        continuous(g_sin_nt_sq(k.suc))
        g_sin_nt_sq_derivative(k)
        is_derivative_fn(g_sin_nt_sq(k.suc), pointwise_mul(sin_nt(k.suc), sin_nt(k.suc)))
        sin_nt_sq_integrable(k)
        is_integrable(pointwise_mul(sin_nt(k.suc), sin_nt(k.suc)), Real.0, two_pi)
        forall(t: Real) {
            if interval_contains(Real.0, two_pi, t) {
                sin_nt_sq_abs_le_one(k, t)
                (pointwise_mul(sin_nt(k.suc), sin_nt(k.suc), t)).abs <= Real.1
                abs_le_one_imp_lower(pointwise_mul(sin_nt(k.suc), sin_nt(k.suc), t))
                -Real.1 <= pointwise_mul(sin_nt(k.suc), sin_nt(k.suc), t)
            }
        }
        forall(t: Real) {
            if interval_contains(Real.0, two_pi, t) {
                sin_nt_sq_abs_le_one(k, t)
                (pointwise_mul(sin_nt(k.suc), sin_nt(k.suc), t)).abs <= Real.1
                abs_le_one_imp_upper(pointwise_mul(sin_nt(k.suc), sin_nt(k.suc), t))
                pointwise_mul(sin_nt(k.suc), sin_nt(k.suc), t) <= Real.1
            }
        }
        Real.0 <= two_pi
        (Real.0 <= two_pi) and continuous(g_sin_nt_sq(k.suc))
        ((Real.0 <= two_pi) and continuous(g_sin_nt_sq(k.suc))) and is_derivative_fn(g_sin_nt_sq(k.suc), pointwise_mul(sin_nt(k.suc), sin_nt(k.suc)))

        (((Real.0 <= two_pi) and continuous(g_sin_nt_sq(k.suc))) and is_derivative_fn(g_sin_nt_sq(k.suc), pointwise_mul(sin_nt(k.suc), sin_nt(k.suc)))) and is_integrable(pointwise_mul(sin_nt(k.suc), sin_nt(k.suc)), Real.0, two_pi)


        ((((Real.0 <= two_pi) and continuous(g_sin_nt_sq(k.suc))) and is_derivative_fn(g_sin_nt_sq(k.suc), pointwise_mul(sin_nt(k.suc), sin_nt(k.suc)))) and is_integrable(pointwise_mul(sin_nt(k.suc), sin_nt(k.suc)), Real.0, two_pi)) and forall(t: Real) {


                interval_contains(Real.0, two_pi, t) implies -Real.1 <= pointwise_mul(sin_nt(k.suc), sin_nt(k.suc), t)
            }
        (((((Real.0 <= two_pi) and continuous(g_sin_nt_sq(k.suc))) and is_derivative_fn(g_sin_nt_sq(k.suc), pointwise_mul(sin_nt(k.suc), sin_nt(k.suc)))) and is_integrable(pointwise_mul(sin_nt(k.suc), sin_nt(k.suc)), Real.0, two_pi)) and forall(t: Real) {


                interval_contains(Real.0, two_pi, t) implies -Real.1 <= pointwise_mul(sin_nt(k.suc), sin_nt(k.suc), t)
            }) and forall(t: Real) {
                interval_contains(Real.0, two_pi, t) implies pointwise_mul(sin_nt(k.suc), sin_nt(k.suc), t) <= Real.1
            }
        if (((((Real.0 <= two_pi) and continuous(g_sin_nt_sq(k.suc))) and is_derivative_fn(g_sin_nt_sq(k.suc), pointwise_mul(sin_nt(k.suc), sin_nt(k.suc)))) and is_integrable(pointwise_mul(sin_nt(k.suc), sin_nt(k.suc)), Real.0, two_pi)) and forall(t: Real) {


                interval_contains(Real.0, two_pi, t) implies -Real.1 <= pointwise_mul(sin_nt(k.suc), sin_nt(k.suc), t)
            }) and forall(t: Real) {
                interval_contains(Real.0, two_pi, t) implies pointwise_mul(sin_nt(k.suc), sin_nt(k.suc), t) <= Real.1
            } {
            ftc2_general(pointwise_mul(sin_nt(k.suc), sin_nt(k.suc)), g_sin_nt_sq(k.suc), Real.0, two_pi, -Real.1, Real.1)

            integral(pointwise_mul(sin_nt(k.suc), sin_nt(k.suc)), Real.0, two_pi) = g_sin_nt_sq(k.suc)(two_pi) - g_sin_nt_sq(k.suc)(Real.0)

            pointwise_add_apply(pointwise_mul(constant[Real, Real](Real.one_half), identity_fn[Real]), pointwise_neg(pointwise_mul(constant[Real, Real](Real.one_half / (two * from_nat[Real](k.suc))), sin_nt(Nat.2 * k.suc))), two_pi)


            pointwise_add(pointwise_mul(constant[Real, Real](Real.one_half), identity_fn[Real]), pointwise_neg(pointwise_mul(constant[Real, Real](Real.one_half / (two * from_nat[Real](k.suc))), sin_nt(Nat.2 * k.suc))), two_pi) = pointwise_mul(constant[Real, Real](Real.one_half), identity_fn[Real], two_pi) +



                    pointwise_neg(pointwise_mul(constant[Real, Real](Real.one_half / (two * from_nat[Real](k.suc))), sin_nt(Nat.2 * k.suc)), two_pi)

            pointwise_mul_apply(constant[Real, Real](Real.one_half), identity_fn[Real], two_pi)
            pointwise_mul(constant[Real, Real](Real.one_half), identity_fn[Real], two_pi) = constant[Real, Real](Real.one_half, two_pi) * identity_fn[Real](two_pi)

            constant[Real, Real](Real.one_half, two_pi) = Real.one_half
            identity_fn[Real](two_pi) = two_pi
            pointwise_mul(constant[Real, Real](Real.one_half), identity_fn[Real], two_pi) = Real.one_half * two_pi
            two_pi = pi + pi
            pi + pi = two * pi
            add_self_two(pi)
            two_pi = two * pi
            Real.one_half * two_pi = Real.one_half * (two * pi)
            Real.one_half = Real.1 / two
            Real.one_half * (two * pi) = (Real.1 / two) * (two * pi)
            half_twice(pi)
            (Real.1 / two) * (two * pi) = pi
            Real.one_half * (two * pi) = pi
            Real.one_half * two_pi = pi
            pointwise_neg_apply(pointwise_mul(constant[Real, Real](Real.one_half / (two * from_nat[Real](k.suc))), sin_nt(Nat.2 * k.suc)), two_pi)

            pointwise_neg(pointwise_mul(constant[Real, Real](Real.one_half / (two * from_nat[Real](k.suc))), sin_nt(Nat.2 * k.suc)), two_pi) = -(pointwise_mul(constant[Real, Real](Real.one_half / (two * from_nat[Real](k.suc))), sin_nt(Nat.2 * k.suc), two_pi))


            pointwise_mul_apply(constant[Real, Real](Real.one_half / (two * from_nat[Real](k.suc))), sin_nt(Nat.2 * k.suc), two_pi)

            pointwise_mul(constant[Real, Real](Real.one_half / (two * from_nat[Real](k.suc))), sin_nt(Nat.2 * k.suc), two_pi) = constant[Real, Real](Real.one_half / (two * from_nat[Real](k.suc)), two_pi) *


                    sin_nt(Nat.2 * k.suc, two_pi)
            constant[Real, Real](Real.one_half / (two * from_nat[Real](k.suc)), two_pi) = Real.one_half / (two * from_nat[Real](k.suc))

            pointwise_mul(constant[Real, Real](Real.one_half / (two * from_nat[Real](k.suc))), sin_nt(Nat.2 * k.suc), two_pi) = (Real.one_half / (two * from_nat[Real](k.suc))) * sin_nt(Nat.2 * k.suc, two_pi)


            sin_nt_two_pi_zero(Nat.2 * k.suc)
            sin_nt(Nat.2 * k.suc, two_pi) = Real.0
            (Real.one_half / (two * from_nat[Real](k.suc))) * sin_nt(Nat.2 * k.suc, two_pi) = Real.0
            pointwise_mul(constant[Real, Real](Real.one_half / (two * from_nat[Real](k.suc))), sin_nt(Nat.2 * k.suc), two_pi) = Real.0

            pointwise_neg(pointwise_mul(constant[Real, Real](Real.one_half / (two * from_nat[Real](k.suc))), sin_nt(Nat.2 * k.suc)), two_pi) = Real.0

            pointwise_add(pointwise_mul(constant[Real, Real](Real.one_half), identity_fn[Real]), pointwise_neg(pointwise_mul(constant[Real, Real](Real.one_half / (two * from_nat[Real](k.suc))), sin_nt(Nat.2 * k.suc))), two_pi) = pi


            g_sin_nt_sq(k.suc)(two_pi) = pi
            pointwise_add_apply(pointwise_mul(constant[Real, Real](Real.one_half), identity_fn[Real]), pointwise_neg(pointwise_mul(constant[Real, Real](Real.one_half / (two * from_nat[Real](k.suc))), sin_nt(Nat.2 * k.suc))), Real.0)


            pointwise_add(pointwise_mul(constant[Real, Real](Real.one_half), identity_fn[Real]), pointwise_neg(pointwise_mul(constant[Real, Real](Real.one_half / (two * from_nat[Real](k.suc))), sin_nt(Nat.2 * k.suc))), Real.0) = pointwise_mul(constant[Real, Real](Real.one_half), identity_fn[Real], Real.0) +



                    pointwise_neg(pointwise_mul(constant[Real, Real](Real.one_half / (two * from_nat[Real](k.suc))), sin_nt(Nat.2 * k.suc)), Real.0)

            pointwise_mul_apply(constant[Real, Real](Real.one_half), identity_fn[Real], Real.0)
            pointwise_mul(constant[Real, Real](Real.one_half), identity_fn[Real], Real.0) = constant[Real, Real](Real.one_half, Real.0) * identity_fn[Real](Real.0)

            constant[Real, Real](Real.one_half, Real.0) = Real.one_half
            identity_fn[Real](Real.0) = Real.0
            pointwise_mul(constant[Real, Real](Real.one_half), identity_fn[Real], Real.0) = Real.one_half * Real.0

            Real.one_half * Real.0 = Real.0
            pointwise_neg_apply(pointwise_mul(constant[Real, Real](Real.one_half / (two * from_nat[Real](k.suc))), sin_nt(Nat.2 * k.suc)), Real.0)

            pointwise_neg(pointwise_mul(constant[Real, Real](Real.one_half / (two * from_nat[Real](k.suc))), sin_nt(Nat.2 * k.suc)), Real.0) = -(pointwise_mul(constant[Real, Real](Real.one_half / (two * from_nat[Real](k.suc))), sin_nt(Nat.2 * k.suc), Real.0))


            pointwise_mul_apply(constant[Real, Real](Real.one_half / (two * from_nat[Real](k.suc))), sin_nt(Nat.2 * k.suc), Real.0)

            pointwise_mul(constant[Real, Real](Real.one_half / (two * from_nat[Real](k.suc))), sin_nt(Nat.2 * k.suc), Real.0) = constant[Real, Real](Real.one_half / (two * from_nat[Real](k.suc)), Real.0) *


                    sin_nt(Nat.2 * k.suc, Real.0)
            constant[Real, Real](Real.one_half / (two * from_nat[Real](k.suc)), Real.0) = Real.one_half / (two * from_nat[Real](k.suc))

            pointwise_mul(constant[Real, Real](Real.one_half / (two * from_nat[Real](k.suc))), sin_nt(Nat.2 * k.suc), Real.0) = (Real.one_half / (two * from_nat[Real](k.suc))) * sin_nt(Nat.2 * k.suc, Real.0)


            sin_nt_at_zero(Nat.2 * k.suc)
            sin_nt(Nat.2 * k.suc, Real.0) = Real.0
            (Real.one_half / (two * from_nat[Real](k.suc))) * sin_nt(Nat.2 * k.suc, Real.0) = Real.0
            pointwise_mul(constant[Real, Real](Real.one_half / (two * from_nat[Real](k.suc))), sin_nt(Nat.2 * k.suc), Real.0) = Real.0

            pointwise_neg(pointwise_mul(constant[Real, Real](Real.one_half / (two * from_nat[Real](k.suc))), sin_nt(Nat.2 * k.suc)), Real.0) = Real.0

            pointwise_add(pointwise_mul(constant[Real, Real](Real.one_half), identity_fn[Real]), pointwise_neg(pointwise_mul(constant[Real, Real](Real.one_half / (two * from_nat[Real](k.suc))), sin_nt(Nat.2 * k.suc))), Real.0) = Real.0


            g_sin_nt_sq(k.suc)(Real.0) = Real.0
            g_sin_nt_sq(k.suc)(two_pi) - g_sin_nt_sq(k.suc)(Real.0) = pi - Real.0
            pi - Real.0 = pi
            integral(pointwise_mul(sin_nt(k.suc), sin_nt(k.suc)), Real.0, two_pi) = pi
        }
}

/// The antiderivative of kx.cos^2: x -> x/2 + (2kx).sin/(4k).
define g_cos_nt_sq(k: Nat) -> (Real -> Real) {
    pointwise_add(pointwise_mul(constant[Real, Real](Real.one_half), identity_fn[Real]), pointwise_mul(constant[Real, Real](Real.one_half / (two * from_nat[Real](k))), sin_nt(Nat.2 * k)))


}

/// The antiderivative of kx.sin kx.cos: x -> -(2kx).cos/(4k).
define g_sin_cos(k: Nat) -> (Real -> Real) {
    pointwise_neg(pointwise_mul(constant[Real, Real](Real.one_half / (two * from_nat[Real](k))), cos_nt(Nat.2 * k)))

}

/// The half-angle identity for the cosine harmonic: kx.cos^2 = (1 + (2kx).cos)/2.
theorem cos_nt_sq_half_angle(k: Nat, x: Real) {
    cos_nt(k.suc, x) * cos_nt(k.suc, x) = (Real.1 + cos_nt(Nat.2 * k.suc, x)) * Real.one_half
} by {
    cos_sq_times_two(from_nat[Real](k.suc) * x)
    two * ((from_nat[Real](k.suc) * x).cos * (from_nat[Real](k.suc) * x).cos) = Real.1 + (two * (from_nat[Real](k.suc) * x)).cos

    cos_nt(k.suc, x) = (from_nat[Real](k.suc) * x).cos
    (from_nat[Real](k.suc) * x).cos = cos_nt(k.suc, x)
    (from_nat[Real](k.suc) * x).cos * (from_nat[Real](k.suc) * x).cos = cos_nt(k.suc, x) * cos_nt(k.suc, x)

    two * ((from_nat[Real](k.suc) * x).cos * (from_nat[Real](k.suc) * x).cos) = two * (cos_nt(k.suc, x) * cos_nt(k.suc, x))

    two * (cos_nt(k.suc, x) * cos_nt(k.suc, x)) = Real.1 + (two * (from_nat[Real](k.suc) * x)).cos
    mul_assoc(two, from_nat[Real](k.suc), x)
    two * (from_nat[Real](k.suc) * x) = (two * from_nat[Real](k.suc)) * x
    (two * (from_nat[Real](k.suc) * x)).cos = ((two * from_nat[Real](k.suc)) * x).cos
    two * (cos_nt(k.suc, x) * cos_nt(k.suc, x)) = Real.1 + ((two * from_nat[Real](k.suc)) * x).cos
    from_nat_two_mul(k.suc)
    from_nat[Real](Nat.2 * k.suc) = two * from_nat[Real](k.suc)
    two * from_nat[Real](k.suc) = from_nat[Real](Nat.2 * k.suc)
    (two * from_nat[Real](k.suc)) * x = from_nat[Real](Nat.2 * k.suc) * x
    ((two * from_nat[Real](k.suc)) * x).cos = (from_nat[Real](Nat.2 * k.suc) * x).cos
    cos_nt(Nat.2 * k.suc, x) = (from_nat[Real](Nat.2 * k.suc) * x).cos
    (from_nat[Real](Nat.2 * k.suc) * x).cos = cos_nt(Nat.2 * k.suc, x)
    two * (cos_nt(k.suc, x) * cos_nt(k.suc, x)) = Real.1 + cos_nt(Nat.2 * k.suc, x)
    two_nonzero
    two != Real.0
    mul_div_cancel(Real.1, two)
    two != Real.0 implies two * (Real.1 / two) = Real.1
    two * (Real.1 / two) = Real.1
    (Real.1 / two) * two = Real.1
    real_mul_comm(Real.one_half, two * (cos_nt(k.suc, x) * cos_nt(k.suc, x)))
    Real.one_half * (two * (cos_nt(k.suc, x) * cos_nt(k.suc, x))) = (two * (cos_nt(k.suc, x) * cos_nt(k.suc, x))) * Real.one_half

    Real.one_half = Real.1 / two
    (Real.1 / two) * (two * (cos_nt(k.suc, x) * cos_nt(k.suc, x))) = (two * (cos_nt(k.suc, x) * cos_nt(k.suc, x))) * Real.one_half

    half_twice(cos_nt(k.suc, x) * cos_nt(k.suc, x))
    (Real.1 / two) * (two * (cos_nt(k.suc, x) * cos_nt(k.suc, x))) = cos_nt(k.suc, x) * cos_nt(k.suc, x)
    cos_nt(k.suc, x) * cos_nt(k.suc, x) = (two * (cos_nt(k.suc, x) * cos_nt(k.suc, x))) * Real.one_half

    Real.one_half * (two * (cos_nt(k.suc, x) * cos_nt(k.suc, x))) = cos_nt(k.suc, x) * cos_nt(k.suc, x)

    (two * (cos_nt(k.suc, x) * cos_nt(k.suc, x))) * Real.one_half = (Real.1 + cos_nt(Nat.2 * k.suc, x)) * Real.one_half

    cos_nt(k.suc, x) * cos_nt(k.suc, x) = (Real.1 + cos_nt(Nat.2 * k.suc, x)) * Real.one_half
}

/// kx.sin kx.cos = (2kx).sin/2.
theorem sin_nt_cos_nt_sin_double(k: Nat, x: Real) {
    sin_nt(k.suc, x) * cos_nt(k.suc, x) = sin_nt(Nat.2 * k.suc, x) * Real.one_half
} by {
    sin_double(from_nat[Real](k.suc) * x)
    (two * (from_nat[Real](k.suc) * x)).sin = two * (from_nat[Real](k.suc) * x).sin * (from_nat[Real](k.suc) * x).cos

    mul_assoc(two, from_nat[Real](k.suc), x)
    two * (from_nat[Real](k.suc) * x) = (two * from_nat[Real](k.suc)) * x
    (two * (from_nat[Real](k.suc) * x)).sin = ((two * from_nat[Real](k.suc)) * x).sin
    ((two * from_nat[Real](k.suc)) * x).sin = two * (from_nat[Real](k.suc) * x).sin * (from_nat[Real](k.suc) * x).cos

    from_nat_two_mul(k.suc)
    from_nat[Real](Nat.2 * k.suc) = two * from_nat[Real](k.suc)
    two * from_nat[Real](k.suc) = from_nat[Real](Nat.2 * k.suc)
    (two * from_nat[Real](k.suc)) * x = from_nat[Real](Nat.2 * k.suc) * x
    ((two * from_nat[Real](k.suc)) * x).sin = (from_nat[Real](Nat.2 * k.suc) * x).sin
    sin_nt(Nat.2 * k.suc, x) = (from_nat[Real](Nat.2 * k.suc) * x).sin
    (from_nat[Real](Nat.2 * k.suc) * x).sin = sin_nt(Nat.2 * k.suc, x)
    sin_nt(Nat.2 * k.suc, x) = two * (from_nat[Real](k.suc) * x).sin * (from_nat[Real](k.suc) * x).cos
    sin_nt(k.suc, x) = (from_nat[Real](k.suc) * x).sin
    (from_nat[Real](k.suc) * x).sin = sin_nt(k.suc, x)
    cos_nt(k.suc, x) = (from_nat[Real](k.suc) * x).cos
    (from_nat[Real](k.suc) * x).cos = cos_nt(k.suc, x)
    two * (from_nat[Real](k.suc) * x).sin * (from_nat[Real](k.suc) * x).cos = two * sin_nt(k.suc, x) * cos_nt(k.suc, x)

    sin_nt(Nat.2 * k.suc, x) = two * sin_nt(k.suc, x) * cos_nt(k.suc, x)
    mul_assoc(two, sin_nt(k.suc, x), cos_nt(k.suc, x))
    two * (sin_nt(k.suc, x) * cos_nt(k.suc, x)) = two * sin_nt(k.suc, x) * cos_nt(k.suc, x)
    sin_nt(Nat.2 * k.suc, x) = two * (sin_nt(k.suc, x) * cos_nt(k.suc, x))
    Real.one_half = Real.1 / two
    two_nonzero
    two != Real.0
    mul_div_cancel(Real.1, two)
    two != Real.0 implies two * (Real.1 / two) = Real.1
    two * (Real.1 / two) = Real.1
    (Real.1 / two) * two = Real.1
    half_twice(sin_nt(k.suc, x) * cos_nt(k.suc, x))
    (Real.1 / two) * (two * (sin_nt(k.suc, x) * cos_nt(k.suc, x))) = sin_nt(k.suc, x) * cos_nt(k.suc, x)
    real_mul_comm(Real.one_half, two * (sin_nt(k.suc, x) * cos_nt(k.suc, x)))
    Real.one_half * (two * (sin_nt(k.suc, x) * cos_nt(k.suc, x))) = (two * (sin_nt(k.suc, x) * cos_nt(k.suc, x))) * Real.one_half

    (Real.1 / two) * (two * (sin_nt(k.suc, x) * cos_nt(k.suc, x))) = Real.one_half * (two * (sin_nt(k.suc, x) * cos_nt(k.suc, x)))

    Real.one_half * (two * (sin_nt(k.suc, x) * cos_nt(k.suc, x))) = sin_nt(k.suc, x) * cos_nt(k.suc, x)

    (two * (sin_nt(k.suc, x) * cos_nt(k.suc, x))) * Real.one_half = sin_nt(k.suc, x) * cos_nt(k.suc, x)

    sin_nt(Nat.2 * k.suc, x) * Real.one_half = sin_nt(k.suc, x) * cos_nt(k.suc, x)
    sin_nt(k.suc, x) * cos_nt(k.suc, x) = sin_nt(Nat.2 * k.suc, x) * Real.one_half
}

/// The derivative of the kx.cos^2 antiderivative is kx.cos^2.
theorem g_cos_nt_sq_derivative(k: Nat) {
    is_derivative_fn(g_cos_nt_sq(k.suc), pointwise_mul(cos_nt(k.suc), cos_nt(k.suc)))
} by {
    sin_nt_derivative(Nat.2 * k.suc)
    is_derivative_fn(sin_nt(Nat.2 * k.suc), pointwise_mul(constant[Real, Real](from_nat[Real](Nat.2 * k.suc)), cos_nt(Nat.2 * k.suc)))

    derivative_fn_const_mul(Real.one_half / (two * from_nat[Real](k.suc)), sin_nt(Nat.2 * k.suc), pointwise_mul(constant[Real, Real](from_nat[Real](Nat.2 * k.suc)), cos_nt(Nat.2 * k.suc)))


    is_derivative_fn(pointwise_mul(constant[Real, Real](Real.one_half / (two * from_nat[Real](k.suc))), sin_nt(Nat.2 * k.suc)), pointwise_mul(constant[Real, Real](Real.one_half / (two * from_nat[Real](k.suc))), pointwise_mul(constant[Real, Real](from_nat[Real](Nat.2 * k.suc)), cos_nt(Nat.2 * k.suc))))



    forall(x: Real) {
        pointwise_mul_apply(constant[Real, Real](Real.one_half / (two * from_nat[Real](k.suc))), pointwise_mul(constant[Real, Real](from_nat[Real](Nat.2 * k.suc)), cos_nt(Nat.2 * k.suc)), x)

        pointwise_mul(constant[Real, Real](Real.one_half / (two * from_nat[Real](k.suc))), pointwise_mul(constant[Real, Real](from_nat[Real](Nat.2 * k.suc)), cos_nt(Nat.2 * k.suc)), x) = constant[Real, Real](Real.one_half / (two * from_nat[Real](k.suc)), x) *


                pointwise_mul(constant[Real, Real](from_nat[Real](Nat.2 * k.suc)), cos_nt(Nat.2 * k.suc), x)
        constant[Real, Real](Real.one_half / (two * from_nat[Real](k.suc)), x) = Real.one_half / (two * from_nat[Real](k.suc))

        pointwise_mul_apply(constant[Real, Real](from_nat[Real](Nat.2 * k.suc)), cos_nt(Nat.2 * k.suc), x)
        pointwise_mul(constant[Real, Real](from_nat[Real](Nat.2 * k.suc)), cos_nt(Nat.2 * k.suc), x) = constant[Real, Real](from_nat[Real](Nat.2 * k.suc), x) * cos_nt(Nat.2 * k.suc, x)

        constant[Real, Real](from_nat[Real](Nat.2 * k.suc), x) = from_nat[Real](Nat.2 * k.suc)
        pointwise_mul(constant[Real, Real](from_nat[Real](Nat.2 * k.suc)), cos_nt(Nat.2 * k.suc), x) = from_nat[Real](Nat.2 * k.suc) * cos_nt(Nat.2 * k.suc, x)

        pointwise_mul(constant[Real, Real](Real.one_half / (two * from_nat[Real](k.suc))), pointwise_mul(constant[Real, Real](from_nat[Real](Nat.2 * k.suc)), cos_nt(Nat.2 * k.suc)), x) = (Real.one_half / (two * from_nat[Real](k.suc))) *


                (from_nat[Real](Nat.2 * k.suc) * cos_nt(Nat.2 * k.suc, x))
        from_nat_two_mul(k.suc)
        from_nat[Real](Nat.2 * k.suc) = two * from_nat[Real](k.suc)
        norm_coeff_times_two_k(k)
        (Real.one_half / (two * from_nat[Real](k.suc))) * (two * from_nat[Real](k.suc)) = Real.one_half
        mul_assoc(Real.one_half / (two * from_nat[Real](k.suc)), two * from_nat[Real](k.suc), cos_nt(Nat.2 * k.suc, x))

        (Real.one_half / (two * from_nat[Real](k.suc))) *
            ((two * from_nat[Real](k.suc)) * cos_nt(Nat.2 * k.suc, x)) = ((Real.one_half / (two * from_nat[Real](k.suc))) * (two * from_nat[Real](k.suc))) *

                cos_nt(Nat.2 * k.suc, x)
        ((Real.one_half / (two * from_nat[Real](k.suc))) * (two * from_nat[Real](k.suc))) *
            cos_nt(Nat.2 * k.suc, x) = Real.one_half * cos_nt(Nat.2 * k.suc, x)
        pointwise_mul(constant[Real, Real](Real.one_half / (two * from_nat[Real](k.suc))), pointwise_mul(constant[Real, Real](from_nat[Real](Nat.2 * k.suc)), cos_nt(Nat.2 * k.suc)), x) = Real.one_half * cos_nt(Nat.2 * k.suc, x)


        pointwise_mul_apply(constant[Real, Real](Real.one_half), cos_nt(Nat.2 * k.suc), x)
        pointwise_mul(constant[Real, Real](Real.one_half), cos_nt(Nat.2 * k.suc), x) = constant[Real, Real](Real.one_half, x) * cos_nt(Nat.2 * k.suc, x)

        constant[Real, Real](Real.one_half, x) = Real.one_half
        pointwise_mul(constant[Real, Real](Real.one_half), cos_nt(Nat.2 * k.suc), x) = Real.one_half * cos_nt(Nat.2 * k.suc, x)

        pointwise_mul(constant[Real, Real](Real.one_half / (two * from_nat[Real](k.suc))), pointwise_mul(constant[Real, Real](from_nat[Real](Nat.2 * k.suc)), cos_nt(Nat.2 * k.suc)), x) = pointwise_mul(constant[Real, Real](Real.one_half), cos_nt(Nat.2 * k.suc), x)


    }
    function_extensionality(pointwise_mul(constant[Real, Real](Real.one_half / (two * from_nat[Real](k.suc))), pointwise_mul(constant[Real, Real](from_nat[Real](Nat.2 * k.suc)), cos_nt(Nat.2 * k.suc))), pointwise_mul(constant[Real, Real](Real.one_half), cos_nt(Nat.2 * k.suc)))


    pointwise_mul(constant[Real, Real](Real.one_half / (two * from_nat[Real](k.suc))), pointwise_mul(constant[Real, Real](from_nat[Real](Nat.2 * k.suc)), cos_nt(Nat.2 * k.suc))) = pointwise_mul(constant[Real, Real](Real.one_half), cos_nt(Nat.2 * k.suc))


    is_derivative_fn(pointwise_mul(constant[Real, Real](Real.one_half / (two * from_nat[Real](k.suc))), sin_nt(Nat.2 * k.suc)), pointwise_mul(constant[Real, Real](Real.one_half), cos_nt(Nat.2 * k.suc)))


    derivative_fn_const_mul(Real.one_half, identity_fn[Real], constant[Real, Real](Real.1))
    is_derivative_fn(pointwise_mul(constant[Real, Real](Real.one_half), identity_fn[Real]), pointwise_mul(constant[Real, Real](Real.one_half), constant[Real, Real](Real.1)))

    forall(x: Real) {
        pointwise_mul_apply(constant[Real, Real](Real.one_half), constant[Real, Real](Real.1), x)
        pointwise_mul(constant[Real, Real](Real.one_half), constant[Real, Real](Real.1), x) = constant[Real, Real](Real.one_half, x) * constant[Real, Real](Real.1, x)

        constant[Real, Real](Real.one_half, x) = Real.one_half
        constant[Real, Real](Real.1, x) = Real.1
        pointwise_mul(constant[Real, Real](Real.one_half), constant[Real, Real](Real.1), x) = Real.one_half * Real.1

        Real.one_half * Real.1 = Real.one_half
        pointwise_mul(constant[Real, Real](Real.one_half), constant[Real, Real](Real.1), x) = Real.one_half
        constant[Real, Real](Real.one_half, x) = Real.one_half
        pointwise_mul(constant[Real, Real](Real.one_half), constant[Real, Real](Real.1), x) = constant[Real, Real](Real.one_half, x)

    }
    function_extensionality(pointwise_mul(constant[Real, Real](Real.one_half), constant[Real, Real](Real.1)), constant[Real, Real](Real.one_half))

    pointwise_mul(constant[Real, Real](Real.one_half), constant[Real, Real](Real.1)) = constant[Real, Real](Real.one_half)

    is_derivative_fn(pointwise_mul(constant[Real, Real](Real.one_half), identity_fn[Real]), constant[Real, Real](Real.one_half))

    derivative_fn_add(pointwise_mul(constant[Real, Real](Real.one_half), identity_fn[Real]), pointwise_mul(constant[Real, Real](Real.one_half / (two * from_nat[Real](k.suc))), sin_nt(Nat.2 * k.suc)), constant[Real, Real](Real.one_half), pointwise_mul(constant[Real, Real](Real.one_half), cos_nt(Nat.2 * k.suc)))




    is_derivative_fn(g_cos_nt_sq(k.suc), pointwise_add(constant[Real, Real](Real.one_half), pointwise_mul(constant[Real, Real](Real.one_half), cos_nt(Nat.2 * k.suc))))


    forall(x: Real) {
        pointwise_add_apply(constant[Real, Real](Real.one_half), pointwise_mul(constant[Real, Real](Real.one_half), cos_nt(Nat.2 * k.suc)), x)

        pointwise_add(constant[Real, Real](Real.one_half), pointwise_mul(constant[Real, Real](Real.one_half), cos_nt(Nat.2 * k.suc)), x) = constant[Real, Real](Real.one_half, x) +


                pointwise_mul(constant[Real, Real](Real.one_half), cos_nt(Nat.2 * k.suc), x)
        constant[Real, Real](Real.one_half, x) = Real.one_half
        pointwise_mul_apply(constant[Real, Real](Real.one_half), cos_nt(Nat.2 * k.suc), x)
        pointwise_mul(constant[Real, Real](Real.one_half), cos_nt(Nat.2 * k.suc), x) = constant[Real, Real](Real.one_half, x) * cos_nt(Nat.2 * k.suc, x)

        constant[Real, Real](Real.one_half, x) = Real.one_half
        pointwise_mul(constant[Real, Real](Real.one_half), cos_nt(Nat.2 * k.suc), x) = Real.one_half * cos_nt(Nat.2 * k.suc, x)

        pointwise_add(constant[Real, Real](Real.one_half), pointwise_mul(constant[Real, Real](Real.one_half), cos_nt(Nat.2 * k.suc)), x) = Real.one_half + Real.one_half * cos_nt(Nat.2 * k.suc, x)


        Real.one_half + Real.one_half * cos_nt(Nat.2 * k.suc, x) = (Real.1 + cos_nt(Nat.2 * k.suc, x)) * Real.one_half

        cos_nt_sq_half_angle(k, x)
        cos_nt(k.suc, x) * cos_nt(k.suc, x) = (Real.1 + cos_nt(Nat.2 * k.suc, x)) * Real.one_half
        pointwise_add(constant[Real, Real](Real.one_half), pointwise_mul(constant[Real, Real](Real.one_half), cos_nt(Nat.2 * k.suc)), x) = cos_nt(k.suc, x) * cos_nt(k.suc, x)


        pointwise_mul_apply(cos_nt(k.suc), cos_nt(k.suc), x)
        pointwise_mul(cos_nt(k.suc), cos_nt(k.suc), x) = cos_nt(k.suc, x) * cos_nt(k.suc, x)
        pointwise_add(constant[Real, Real](Real.one_half), pointwise_mul(constant[Real, Real](Real.one_half), cos_nt(Nat.2 * k.suc)), x) = pointwise_mul(cos_nt(k.suc), cos_nt(k.suc), x)


    }
    function_extensionality(pointwise_add(constant[Real, Real](Real.one_half), pointwise_mul(constant[Real, Real](Real.one_half), cos_nt(Nat.2 * k.suc))), pointwise_mul(cos_nt(k.suc), cos_nt(k.suc)))


    pointwise_add(constant[Real, Real](Real.one_half), pointwise_mul(constant[Real, Real](Real.one_half), cos_nt(Nat.2 * k.suc))) = pointwise_mul(cos_nt(k.suc), cos_nt(k.suc))


    is_derivative_fn(g_cos_nt_sq(k.suc), pointwise_mul(cos_nt(k.suc), cos_nt(k.suc)))
}

/// The antiderivative g_cos_nt_sq(k.suc) is continuous.
theorem g_cos_nt_sq_continuous(k: Nat) {
    continuous(g_cos_nt_sq(k.suc))
} by {
    constant_function_is_continuous(Real.one_half)
    continuous(constant[Real, Real](Real.one_half))
    identity_function_is_continuous
    continuous(identity_fn[Real])
    continuous_pointwise_mul(constant[Real, Real](Real.one_half), identity_fn[Real])
    continuous(constant[Real, Real](Real.one_half)) and continuous(identity_fn[Real]) implies continuous(pointwise_mul(constant[Real, Real](Real.one_half), identity_fn[Real]))

    continuous(pointwise_mul(constant[Real, Real](Real.one_half), identity_fn[Real]))
    constant_function_is_continuous(Real.one_half / (two * from_nat[Real](k.suc)))
    continuous(constant[Real, Real](Real.one_half / (two * from_nat[Real](k.suc))))
    sin_nt_continuous(Nat.2 * k.suc)
    continuous(sin_nt(Nat.2 * k.suc))
    continuous_pointwise_mul(constant[Real, Real](Real.one_half / (two * from_nat[Real](k.suc))), sin_nt(Nat.2 * k.suc))

    continuous(constant[Real, Real](Real.one_half / (two * from_nat[Real](k.suc)))) and continuous(sin_nt(Nat.2 * k.suc)) implies continuous(pointwise_mul(constant[Real, Real](Real.one_half / (two * from_nat[Real](k.suc))), sin_nt(Nat.2 * k.suc)))


    continuous(pointwise_mul(constant[Real, Real](Real.one_half / (two * from_nat[Real](k.suc))), sin_nt(Nat.2 * k.suc)))

    continuous_pointwise_add(pointwise_mul(constant[Real, Real](Real.one_half), identity_fn[Real]), pointwise_mul(constant[Real, Real](Real.one_half / (two * from_nat[Real](k.suc))), sin_nt(Nat.2 * k.suc)))


    continuous(pointwise_mul(constant[Real, Real](Real.one_half), identity_fn[Real])) and continuous(pointwise_mul(constant[Real, Real](Real.one_half / (two * from_nat[Real](k.suc))),  sin_nt(Nat.2 * k.suc))) implies continuous(pointwise_add(pointwise_mul(constant[Real, Real](Real.one_half), identity_fn[Real]), pointwise_mul(constant[Real, Real](Real.one_half / (two * from_nat[Real](k.suc))), sin_nt(Nat.2 * k.suc))))



    continuous(pointwise_add(pointwise_mul(constant[Real, Real](Real.one_half), identity_fn[Real]), pointwise_mul(constant[Real, Real](Real.one_half / (two * from_nat[Real](k.suc))), sin_nt(Nat.2 * k.suc))))


    g_cos_nt_sq(k.suc) = pointwise_add(pointwise_mul(constant[Real, Real](Real.one_half), identity_fn[Real]), pointwise_mul(constant[Real, Real](Real.one_half / (two * from_nat[Real](k.suc))), sin_nt(Nat.2 * k.suc)))


    continuous(g_cos_nt_sq(k.suc))
}

/// The derivative of the Real.sin-Real.cos antiderivative is kx.sin kx.cos.
theorem g_sin_cos_derivative(k: Nat) {
    is_derivative_fn(g_sin_cos(k.suc), pointwise_mul(sin_nt(k.suc), cos_nt(k.suc)))
} by {
    cos_nt_derivative(Nat.2 * k.suc)
    is_derivative_fn(cos_nt(Nat.2 * k.suc), pointwise_neg(pointwise_mul(constant[Real, Real](from_nat[Real](Nat.2 * k.suc)), sin_nt(Nat.2 * k.suc))))

    derivative_fn_const_mul(Real.one_half / (two * from_nat[Real](k.suc)), cos_nt(Nat.2 * k.suc), pointwise_neg(pointwise_mul(constant[Real, Real](from_nat[Real](Nat.2 * k.suc)), sin_nt(Nat.2 * k.suc))))


    is_derivative_fn(pointwise_mul(constant[Real, Real](Real.one_half / (two * from_nat[Real](k.suc))), cos_nt(Nat.2 * k.suc)), pointwise_mul(constant[Real, Real](Real.one_half / (two * from_nat[Real](k.suc))), pointwise_neg(pointwise_mul(constant[Real, Real](from_nat[Real](Nat.2 * k.suc)), sin_nt(Nat.2 * k.suc)))))



    derivative_fn_neg(pointwise_mul(constant[Real, Real](Real.one_half / (two * from_nat[Real](k.suc))), cos_nt(Nat.2 * k.suc)), pointwise_mul(constant[Real, Real](Real.one_half / (two * from_nat[Real](k.suc))), pointwise_neg(pointwise_mul(constant[Real, Real](from_nat[Real](Nat.2 * k.suc)), sin_nt(Nat.2 * k.suc)))))



    is_derivative_fn(pointwise_neg(pointwise_mul(constant[Real, Real](Real.one_half / (two * from_nat[Real](k.suc))), cos_nt(Nat.2 * k.suc))), pointwise_neg(pointwise_mul(constant[Real, Real](Real.one_half / (two * from_nat[Real](k.suc))), pointwise_neg(pointwise_mul(constant[Real, Real](from_nat[Real](Nat.2 * k.suc)), sin_nt(Nat.2 * k.suc))))))



    forall(x: Real) {
        pointwise_neg_apply(pointwise_mul(constant[Real, Real](Real.one_half / (two * from_nat[Real](k.suc))), pointwise_neg(pointwise_mul(constant[Real, Real](from_nat[Real](Nat.2 * k.suc)), sin_nt(Nat.2 * k.suc)))), x)

        pointwise_neg(pointwise_mul(constant[Real, Real](Real.one_half / (two * from_nat[Real](k.suc))), pointwise_neg(pointwise_mul(constant[Real, Real](from_nat[Real](Nat.2 * k.suc)), sin_nt(Nat.2 * k.suc)))), x) = -(pointwise_mul(constant[Real, Real](Real.one_half / (two * from_nat[Real](k.suc))), pointwise_neg(pointwise_mul(constant[Real, Real](from_nat[Real](Nat.2 * k.suc)), sin_nt(Nat.2 * k.suc))), x))


        pointwise_mul_apply(constant[Real, Real](Real.one_half / (two * from_nat[Real](k.suc))), pointwise_neg(pointwise_mul(constant[Real, Real](from_nat[Real](Nat.2 * k.suc)), sin_nt(Nat.2 * k.suc))), x)

        pointwise_mul(constant[Real, Real](Real.one_half / (two * from_nat[Real](k.suc))), pointwise_neg(pointwise_mul(constant[Real, Real](from_nat[Real](Nat.2 * k.suc)), sin_nt(Nat.2 * k.suc))), x) = constant[Real, Real](Real.one_half / (two * from_nat[Real](k.suc)), x) *


                pointwise_neg(pointwise_mul(constant[Real, Real](from_nat[Real](Nat.2 * k.suc)), sin_nt(Nat.2 * k.suc)), x)
        constant[Real, Real](Real.one_half / (two * from_nat[Real](k.suc)), x) = Real.one_half / (two * from_nat[Real](k.suc))

        pointwise_neg_apply(pointwise_mul(constant[Real, Real](from_nat[Real](Nat.2 * k.suc)), sin_nt(Nat.2 * k.suc)), x)
        pointwise_neg(pointwise_mul(constant[Real, Real](from_nat[Real](Nat.2 * k.suc)), sin_nt(Nat.2 * k.suc)), x) = -(pointwise_mul(constant[Real, Real](from_nat[Real](Nat.2 * k.suc)), sin_nt(Nat.2 * k.suc), x))

        pointwise_mul_apply(constant[Real, Real](from_nat[Real](Nat.2 * k.suc)), sin_nt(Nat.2 * k.suc), x)
        pointwise_mul(constant[Real, Real](from_nat[Real](Nat.2 * k.suc)), sin_nt(Nat.2 * k.suc), x) = constant[Real, Real](from_nat[Real](Nat.2 * k.suc), x) * sin_nt(Nat.2 * k.suc, x)

        constant[Real, Real](from_nat[Real](Nat.2 * k.suc), x) = from_nat[Real](Nat.2 * k.suc)
        pointwise_mul(constant[Real, Real](from_nat[Real](Nat.2 * k.suc)), sin_nt(Nat.2 * k.suc), x) = from_nat[Real](Nat.2 * k.suc) * sin_nt(Nat.2 * k.suc, x)

        pointwise_neg(pointwise_mul(constant[Real, Real](from_nat[Real](Nat.2 * k.suc)), sin_nt(Nat.2 * k.suc)), x) = -(from_nat[Real](Nat.2 * k.suc) * sin_nt(Nat.2 * k.suc, x))

        pointwise_mul(constant[Real, Real](Real.one_half / (two * from_nat[Real](k.suc))), pointwise_neg(pointwise_mul(constant[Real, Real](from_nat[Real](Nat.2 * k.suc)), sin_nt(Nat.2 * k.suc))), x) = (Real.one_half / (two * from_nat[Real](k.suc))) * (-(from_nat[Real](Nat.2 * k.suc) * sin_nt(Nat.2 * k.suc, x)))


        pointwise_neg(pointwise_mul(constant[Real, Real](Real.one_half / (two * from_nat[Real](k.suc))), pointwise_neg(pointwise_mul(constant[Real, Real](from_nat[Real](Nat.2 * k.suc)), sin_nt(Nat.2 * k.suc)))), x) = -((Real.one_half / (two * from_nat[Real](k.suc))) * (-(from_nat[Real](Nat.2 * k.suc) * sin_nt(Nat.2 * k.suc, x))))


        from_nat_two_mul(k.suc)
        from_nat[Real](Nat.2 * k.suc) = two * from_nat[Real](k.suc)
        norm_coeff_times_two_k(k)
        (Real.one_half / (two * from_nat[Real](k.suc))) * (two * from_nat[Real](k.suc)) = Real.one_half
        mul_assoc(Real.one_half / (two * from_nat[Real](k.suc)), two * from_nat[Real](k.suc), sin_nt(Nat.2 * k.suc, x))

        (Real.one_half / (two * from_nat[Real](k.suc))) *
            ((two * from_nat[Real](k.suc)) * sin_nt(Nat.2 * k.suc, x)) = ((Real.one_half / (two * from_nat[Real](k.suc))) * (two * from_nat[Real](k.suc))) *

                sin_nt(Nat.2 * k.suc, x)
        ((Real.one_half / (two * from_nat[Real](k.suc))) * (two * from_nat[Real](k.suc))) *
            sin_nt(Nat.2 * k.suc, x) = Real.one_half * sin_nt(Nat.2 * k.suc, x)
        (Real.one_half / (two * from_nat[Real](k.suc))) *
            ((two * from_nat[Real](k.suc)) * sin_nt(Nat.2 * k.suc, x)) = Real.one_half * sin_nt(Nat.2 * k.suc, x)

        (Real.one_half / (two * from_nat[Real](k.suc))) *
            (from_nat[Real](Nat.2 * k.suc) * sin_nt(Nat.2 * k.suc, x)) = Real.one_half * sin_nt(Nat.2 * k.suc, x)

        (Real.one_half / (two * from_nat[Real](k.suc))) * (-(from_nat[Real](Nat.2 * k.suc) * sin_nt(Nat.2 * k.suc, x))) = -(Real.one_half * sin_nt(Nat.2 * k.suc, x))

        -((Real.one_half / (two * from_nat[Real](k.suc))) * (-(from_nat[Real](Nat.2 * k.suc) * sin_nt(Nat.2 * k.suc, x)))) = Real.one_half * sin_nt(Nat.2 * k.suc, x)

        pointwise_neg(pointwise_mul(constant[Real, Real](Real.one_half / (two * from_nat[Real](k.suc))), pointwise_neg(pointwise_mul(constant[Real, Real](from_nat[Real](Nat.2 * k.suc)), sin_nt(Nat.2 * k.suc)))), x) = Real.one_half * sin_nt(Nat.2 * k.suc, x)


        pointwise_mul_apply(constant[Real, Real](Real.one_half), sin_nt(Nat.2 * k.suc), x)
        pointwise_mul(constant[Real, Real](Real.one_half), sin_nt(Nat.2 * k.suc), x) = constant[Real, Real](Real.one_half, x) * sin_nt(Nat.2 * k.suc, x)

        constant[Real, Real](Real.one_half, x) = Real.one_half
        pointwise_mul(constant[Real, Real](Real.one_half), sin_nt(Nat.2 * k.suc), x) = Real.one_half * sin_nt(Nat.2 * k.suc, x)

        pointwise_neg(pointwise_mul(constant[Real, Real](Real.one_half / (two * from_nat[Real](k.suc))), pointwise_neg(pointwise_mul(constant[Real, Real](from_nat[Real](Nat.2 * k.suc)), sin_nt(Nat.2 * k.suc)))), x) = pointwise_mul(constant[Real, Real](Real.one_half), sin_nt(Nat.2 * k.suc), x)


    }
    function_extensionality(pointwise_neg(pointwise_mul(constant[Real, Real](Real.one_half / (two * from_nat[Real](k.suc))), pointwise_neg(pointwise_mul(constant[Real, Real](from_nat[Real](Nat.2 * k.suc)), sin_nt(Nat.2 * k.suc))))), pointwise_mul(constant[Real, Real](Real.one_half), sin_nt(Nat.2 * k.suc)))


    pointwise_neg(pointwise_mul(constant[Real, Real](Real.one_half / (two * from_nat[Real](k.suc))), pointwise_neg(pointwise_mul(constant[Real, Real](from_nat[Real](Nat.2 * k.suc)), sin_nt(Nat.2 * k.suc))))) = pointwise_mul(constant[Real, Real](Real.one_half), sin_nt(Nat.2 * k.suc))


    is_derivative_fn(pointwise_neg(pointwise_mul(constant[Real, Real](Real.one_half / (two * from_nat[Real](k.suc))), cos_nt(Nat.2 * k.suc))), pointwise_mul(constant[Real, Real](Real.one_half), sin_nt(Nat.2 * k.suc)))


    g_sin_cos(k.suc) = pointwise_neg(pointwise_mul(constant[Real, Real](Real.one_half / (two * from_nat[Real](k.suc))), cos_nt(Nat.2 * k.suc)))

    is_derivative_fn(g_sin_cos(k.suc), pointwise_mul(constant[Real, Real](Real.one_half), sin_nt(Nat.2 * k.suc)))
    forall(x: Real) {
        pointwise_mul_apply(constant[Real, Real](Real.one_half), sin_nt(Nat.2 * k.suc), x)
        pointwise_mul(constant[Real, Real](Real.one_half), sin_nt(Nat.2 * k.suc), x) = constant[Real, Real](Real.one_half, x) * sin_nt(Nat.2 * k.suc, x)

        constant[Real, Real](Real.one_half, x) = Real.one_half
        pointwise_mul(constant[Real, Real](Real.one_half), sin_nt(Nat.2 * k.suc), x) = Real.one_half * sin_nt(Nat.2 * k.suc, x)

        sin_nt_cos_nt_sin_double(k, x)
        sin_nt(k.suc, x) * cos_nt(k.suc, x) = sin_nt(Nat.2 * k.suc, x) * Real.one_half
        real_mul_comm(sin_nt(Nat.2 * k.suc, x), Real.one_half)
        sin_nt(Nat.2 * k.suc, x) * Real.one_half = Real.one_half * sin_nt(Nat.2 * k.suc, x)
        sin_nt(k.suc, x) * cos_nt(k.suc, x) = Real.one_half * sin_nt(Nat.2 * k.suc, x)
        pointwise_mul(constant[Real, Real](Real.one_half), sin_nt(Nat.2 * k.suc), x) = sin_nt(k.suc, x) * cos_nt(k.suc, x)

        pointwise_mul_apply(sin_nt(k.suc), cos_nt(k.suc), x)
        pointwise_mul(sin_nt(k.suc), cos_nt(k.suc), x) = sin_nt(k.suc, x) * cos_nt(k.suc, x)
        pointwise_mul(constant[Real, Real](Real.one_half), sin_nt(Nat.2 * k.suc), x) = pointwise_mul(sin_nt(k.suc), cos_nt(k.suc), x)

    }
    function_extensionality(pointwise_mul(constant[Real, Real](Real.one_half), sin_nt(Nat.2 * k.suc)), pointwise_mul(sin_nt(k.suc), cos_nt(k.suc)))

    pointwise_mul(constant[Real, Real](Real.one_half), sin_nt(Nat.2 * k.suc)) = pointwise_mul(sin_nt(k.suc), cos_nt(k.suc))

    is_derivative_fn(g_sin_cos(k.suc), pointwise_mul(sin_nt(k.suc), cos_nt(k.suc)))
}

/// The antiderivative g_sin_cos(k.suc) is continuous.
theorem g_sin_cos_continuous(k: Nat) {
    continuous(g_sin_cos(k.suc))
} by {
    constant_function_is_continuous(Real.one_half / (two * from_nat[Real](k.suc)))
    continuous(constant[Real, Real](Real.one_half / (two * from_nat[Real](k.suc))))
    cos_nt_continuous(Nat.2 * k.suc)
    continuous(cos_nt(Nat.2 * k.suc))
    continuous_pointwise_mul(constant[Real, Real](Real.one_half / (two * from_nat[Real](k.suc))), cos_nt(Nat.2 * k.suc))

    continuous(constant[Real, Real](Real.one_half / (two * from_nat[Real](k.suc)))) and continuous(cos_nt(Nat.2 * k.suc)) implies continuous(pointwise_mul(constant[Real, Real](Real.one_half / (two * from_nat[Real](k.suc))), cos_nt(Nat.2 * k.suc)))



    continuous(pointwise_mul(constant[Real, Real](Real.one_half / (two * from_nat[Real](k.suc))), cos_nt(Nat.2 * k.suc)))

    continuous_pointwise_neg(pointwise_mul(constant[Real, Real](Real.one_half / (two * from_nat[Real](k.suc))), cos_nt(Nat.2 * k.suc)))

    continuous(pointwise_neg(pointwise_mul(constant[Real, Real](Real.one_half / (two * from_nat[Real](k.suc))), cos_nt(Nat.2 * k.suc))))

    g_sin_cos(k.suc) = pointwise_neg(pointwise_mul(constant[Real, Real](Real.one_half / (two * from_nat[Real](k.suc))), cos_nt(Nat.2 * k.suc)))

    continuous(g_sin_cos(k.suc))
}

/// The square of the cosine harmonic is bounded by one in absolute value.
theorem cos_nt_sq_abs_le_one(k: Nat, x: Real) {
    (pointwise_mul(cos_nt(k.suc), cos_nt(k.suc), x)).abs <= Real.1
} by {
    pointwise_mul_apply(cos_nt(k.suc), cos_nt(k.suc), x)
    pointwise_mul(cos_nt(k.suc), cos_nt(k.suc), x) = cos_nt(k.suc, x) * cos_nt(k.suc, x)
    (pointwise_mul(cos_nt(k.suc), cos_nt(k.suc), x)).abs = (cos_nt(k.suc, x) * cos_nt(k.suc, x)).abs
    cos_nt_abs_le_one(k.suc, x)
    (cos_nt(k.suc, x)).abs <= Real.1
    abs_mul_le_one(cos_nt(k.suc, x), cos_nt(k.suc, x))
    (cos_nt(k.suc, x)).abs <= Real.1 and (cos_nt(k.suc, x)).abs <= Real.1 implies (cos_nt(k.suc, x) * cos_nt(k.suc, x)).abs <= Real.1

    (cos_nt(k.suc, x) * cos_nt(k.suc, x)).abs <= Real.1
    (pointwise_mul(cos_nt(k.suc), cos_nt(k.suc), x)).abs <= Real.1
}

/// The product of the sine and cosine harmonics is bounded by one.
theorem sin_nt_cos_nt_abs_le_one(k: Nat, x: Real) {
    (pointwise_mul(sin_nt(k.suc), cos_nt(k.suc), x)).abs <= Real.1
} by {
    pointwise_mul_apply(sin_nt(k.suc), cos_nt(k.suc), x)
    pointwise_mul(sin_nt(k.suc), cos_nt(k.suc), x) = sin_nt(k.suc, x) * cos_nt(k.suc, x)
    (pointwise_mul(sin_nt(k.suc), cos_nt(k.suc), x)).abs = (sin_nt(k.suc, x) * cos_nt(k.suc, x)).abs

    sin_nt_abs_le_one(k.suc, x)
    (sin_nt(k.suc, x)).abs <= Real.1
    cos_nt_abs_le_one(k.suc, x)
    (cos_nt(k.suc, x)).abs <= Real.1
    abs_mul_le_one(sin_nt(k.suc, x), cos_nt(k.suc, x))
    (sin_nt(k.suc, x)).abs <= Real.1 and (cos_nt(k.suc, x)).abs <= Real.1 implies (sin_nt(k.suc, x) * cos_nt(k.suc, x)).abs <= Real.1

    (sin_nt(k.suc, x) * cos_nt(k.suc, x)).abs <= Real.1
    (pointwise_mul(sin_nt(k.suc), cos_nt(k.suc), x)).abs <= Real.1
}

/// The square of the cosine harmonic is 2k-Lipschitz.
theorem cos_nt_sq_lipschitz(k: Nat, u: Real, v: Real) {
    (pointwise_mul(cos_nt(k.suc), cos_nt(k.suc), u) - pointwise_mul(cos_nt(k.suc), cos_nt(k.suc), v)).abs <= (two * from_nat[Real](k.suc)) * (u - v).abs

} by {
    pointwise_mul_apply(cos_nt(k.suc), cos_nt(k.suc), u)
    pointwise_mul(cos_nt(k.suc), cos_nt(k.suc), u) = cos_nt(k.suc, u) * cos_nt(k.suc, u)
    pointwise_mul_apply(cos_nt(k.suc), cos_nt(k.suc), v)
    pointwise_mul(cos_nt(k.suc), cos_nt(k.suc), v) = cos_nt(k.suc, v) * cos_nt(k.suc, v)
    pointwise_mul(cos_nt(k.suc), cos_nt(k.suc), u) - pointwise_mul(cos_nt(k.suc), cos_nt(k.suc), v) = cos_nt(k.suc, u) * cos_nt(k.suc, u) - cos_nt(k.suc, v) * cos_nt(k.suc, v)

    mul_sub_distrib_left(cos_nt(k.suc, u), cos_nt(k.suc, v), cos_nt(k.suc, u) + cos_nt(k.suc, v))
    (cos_nt(k.suc, u) - cos_nt(k.suc, v)) * (cos_nt(k.suc, u) + cos_nt(k.suc, v)) = cos_nt(k.suc, u) * (cos_nt(k.suc, u) + cos_nt(k.suc, v)) -

            cos_nt(k.suc, v) * (cos_nt(k.suc, u) + cos_nt(k.suc, v))
    mul_distrib_right(cos_nt(k.suc, u), cos_nt(k.suc, u), cos_nt(k.suc, v))
    cos_nt(k.suc, u) * (cos_nt(k.suc, u) + cos_nt(k.suc, v)) = cos_nt(k.suc, u) * cos_nt(k.suc, u) + cos_nt(k.suc, u) * cos_nt(k.suc, v)

    mul_distrib_right(cos_nt(k.suc, v), cos_nt(k.suc, u), cos_nt(k.suc, v))
    cos_nt(k.suc, v) * (cos_nt(k.suc, u) + cos_nt(k.suc, v)) = cos_nt(k.suc, v) * cos_nt(k.suc, u) + cos_nt(k.suc, v) * cos_nt(k.suc, v)

    (cos_nt(k.suc, u) - cos_nt(k.suc, v)) * (cos_nt(k.suc, u) + cos_nt(k.suc, v)) = (cos_nt(k.suc, u) * cos_nt(k.suc, u) + cos_nt(k.suc, u) * cos_nt(k.suc, v)) -

            (cos_nt(k.suc, v) * cos_nt(k.suc, u) + cos_nt(k.suc, v) * cos_nt(k.suc, v))
    cos_nt(k.suc, u) * cos_nt(k.suc, v) = cos_nt(k.suc, v) * cos_nt(k.suc, u)
    add_sub_add_cancel(cos_nt(k.suc, u) * cos_nt(k.suc, u), cos_nt(k.suc, v) * cos_nt(k.suc, u), cos_nt(k.suc, v) * cos_nt(k.suc, v))

    (cos_nt(k.suc, u) * cos_nt(k.suc, u) + cos_nt(k.suc, v) * cos_nt(k.suc, u)) -
        (cos_nt(k.suc, v) * cos_nt(k.suc, u) + cos_nt(k.suc, v) * cos_nt(k.suc, v)) = cos_nt(k.suc, u) * cos_nt(k.suc, u) - cos_nt(k.suc, v) * cos_nt(k.suc, v)

    (cos_nt(k.suc, u) - cos_nt(k.suc, v)) * (cos_nt(k.suc, u) + cos_nt(k.suc, v)) = cos_nt(k.suc, u) * cos_nt(k.suc, u) - cos_nt(k.suc, v) * cos_nt(k.suc, v)

    (pointwise_mul(cos_nt(k.suc), cos_nt(k.suc), u) - pointwise_mul(cos_nt(k.suc), cos_nt(k.suc), v)).abs <= ((cos_nt(k.suc, u) - cos_nt(k.suc, v)) * (cos_nt(k.suc, u) + cos_nt(k.suc, v))).abs

    mul_abs(cos_nt(k.suc, u) - cos_nt(k.suc, v), cos_nt(k.suc, u) + cos_nt(k.suc, v))
    ((cos_nt(k.suc, u) - cos_nt(k.suc, v)) * (cos_nt(k.suc, u) + cos_nt(k.suc, v))).abs = (cos_nt(k.suc, u) - cos_nt(k.suc, v)).abs * (cos_nt(k.suc, u) + cos_nt(k.suc, v)).abs

    (pointwise_mul(cos_nt(k.suc), cos_nt(k.suc), u) - pointwise_mul(cos_nt(k.suc), cos_nt(k.suc), v)).abs <= (cos_nt(k.suc, u) - cos_nt(k.suc, v)).abs * (cos_nt(k.suc, u) + cos_nt(k.suc, v)).abs

    cos_nt_lipschitz(k.suc, u, v)
    (cos_nt(k.suc, u) - cos_nt(k.suc, v)).abs <= from_nat[Real](k.suc) * (u - v).abs
    abs_gte_zero(cos_nt(k.suc, u) + cos_nt(k.suc, v))
    Real.0 <= (cos_nt(k.suc, u) + cos_nt(k.suc, v)).abs
    mul_le_mul_of_nonneg_right((cos_nt(k.suc, u) - cos_nt(k.suc, v)).abs, from_nat[Real](k.suc) * (u - v).abs, (cos_nt(k.suc, u) + cos_nt(k.suc, v)).abs)

    (cos_nt(k.suc, u) - cos_nt(k.suc, v)).abs * (cos_nt(k.suc, u) + cos_nt(k.suc, v)).abs <= (from_nat[Real](k.suc) * (u - v).abs) * (cos_nt(k.suc, u) + cos_nt(k.suc, v)).abs

    cos_add_abs_le_two(from_nat[Real](k.suc) * u, from_nat[Real](k.suc) * v)
    ((from_nat[Real](k.suc) * u).cos + (from_nat[Real](k.suc) * v).cos).abs <= two
    cos_nt(k.suc, u) = (from_nat[Real](k.suc) * u).cos
    (from_nat[Real](k.suc) * u).cos = cos_nt(k.suc, u)
    cos_nt(k.suc, v) = (from_nat[Real](k.suc) * v).cos
    (from_nat[Real](k.suc) * v).cos = cos_nt(k.suc, v)
    ((from_nat[Real](k.suc) * u).cos + (from_nat[Real](k.suc) * v).cos).abs = (cos_nt(k.suc, u) + cos_nt(k.suc, v)).abs

    (cos_nt(k.suc, u) + cos_nt(k.suc, v)).abs <= two
    from_nat_nonneg_real(k.suc)
    Real.0 <= from_nat[Real](k.suc)
    from_nat[Real](k.suc) >= Real.0
    abs_gte_zero((u - v).abs)
    Real.0 <= (u - v).abs
    (u - v).abs >= Real.0
    mul_nonneg(from_nat[Real](k.suc), (u - v).abs)
    from_nat[Real](k.suc) >= Real.0 and (u - v).abs >= Real.0 implies from_nat[Real](k.suc) * (u - v).abs >= Real.0

    from_nat[Real](k.suc) * (u - v).abs >= Real.0
    Real.0 <= from_nat[Real](k.suc) * (u - v).abs
    mul_le_mul_of_nonneg_left((cos_nt(k.suc, u) + cos_nt(k.suc, v)).abs, two, from_nat[Real](k.suc) * (u - v).abs)

    (from_nat[Real](k.suc) * (u - v).abs) * (cos_nt(k.suc, u) + cos_nt(k.suc, v)).abs <= (from_nat[Real](k.suc) * (u - v).abs) * two

    lte_trans[Real]((cos_nt(k.suc, u) - cos_nt(k.suc, v)).abs * (cos_nt(k.suc, u) + cos_nt(k.suc, v)).abs, (from_nat[Real](k.suc) * (u - v).abs) * (cos_nt(k.suc, u) + cos_nt(k.suc, v)).abs, (from_nat[Real](k.suc) * (u - v).abs) * two)


    (cos_nt(k.suc, u) - cos_nt(k.suc, v)).abs * (cos_nt(k.suc, u) + cos_nt(k.suc, v)).abs <= (from_nat[Real](k.suc) * (u - v).abs) * two

    (from_nat[Real](k.suc) * (u - v).abs) * two = two * (from_nat[Real](k.suc) * (u - v).abs)
    (cos_nt(k.suc, u) - cos_nt(k.suc, v)).abs * (cos_nt(k.suc, u) + cos_nt(k.suc, v)).abs <= two * (from_nat[Real](k.suc) * (u - v).abs)

    mul_assoc(two, from_nat[Real](k.suc), (u - v).abs)
    two * (from_nat[Real](k.suc) * (u - v).abs) = (two * from_nat[Real](k.suc)) * (u - v).abs
    (cos_nt(k.suc, u) - cos_nt(k.suc, v)).abs * (cos_nt(k.suc, u) + cos_nt(k.suc, v)).abs <= (two * from_nat[Real](k.suc)) * (u - v).abs

    lte_trans[Real]((pointwise_mul(cos_nt(k.suc), cos_nt(k.suc), u) - pointwise_mul(cos_nt(k.suc), cos_nt(k.suc), v)).abs, (cos_nt(k.suc, u) - cos_nt(k.suc, v)).abs * (cos_nt(k.suc, u) + cos_nt(k.suc, v)).abs, (two * from_nat[Real](k.suc)) * (u - v).abs)


    (pointwise_mul(cos_nt(k.suc), cos_nt(k.suc), u) - pointwise_mul(cos_nt(k.suc), cos_nt(k.suc), v)).abs <= (two * from_nat[Real](k.suc)) * (u - v).abs

}

/// The product of the sine and cosine harmonics is 2k-Lipschitz.
theorem sin_nt_cos_nt_lipschitz(k: Nat, u: Real, v: Real) {
    (pointwise_mul(sin_nt(k.suc), cos_nt(k.suc), u) - pointwise_mul(sin_nt(k.suc), cos_nt(k.suc), v)).abs <= (two * from_nat[Real](k.suc)) * (u - v).abs

} by {
    pointwise_mul_apply(sin_nt(k.suc), cos_nt(k.suc), u)
    pointwise_mul(sin_nt(k.suc), cos_nt(k.suc), u) = sin_nt(k.suc, u) * cos_nt(k.suc, u)
    pointwise_mul_apply(sin_nt(k.suc), cos_nt(k.suc), v)
    pointwise_mul(sin_nt(k.suc), cos_nt(k.suc), v) = sin_nt(k.suc, v) * cos_nt(k.suc, v)
    pointwise_mul(sin_nt(k.suc), cos_nt(k.suc), u) - pointwise_mul(sin_nt(k.suc), cos_nt(k.suc), v) = sin_nt(k.suc, u) * cos_nt(k.suc, u) - sin_nt(k.suc, v) * cos_nt(k.suc, v)

    sincos_lipschitz_m2(from_nat[Real](k.suc) * u, from_nat[Real](k.suc) * v)
    (sincos(from_nat[Real](k.suc) * u) - sincos(from_nat[Real](k.suc) * v)).abs <= two * (from_nat[Real](k.suc) * u - from_nat[Real](k.suc) * v).abs

    sincos(from_nat[Real](k.suc) * u) = (from_nat[Real](k.suc) * u).sin * (from_nat[Real](k.suc) * u).cos
    sin_nt(k.suc, u) = (from_nat[Real](k.suc) * u).sin
    (from_nat[Real](k.suc) * u).sin = sin_nt(k.suc, u)
    cos_nt(k.suc, u) = (from_nat[Real](k.suc) * u).cos
    (from_nat[Real](k.suc) * u).cos = cos_nt(k.suc, u)
    sincos(from_nat[Real](k.suc) * u) = sin_nt(k.suc, u) * cos_nt(k.suc, u)
    sincos(from_nat[Real](k.suc) * v) = (from_nat[Real](k.suc) * v).sin * (from_nat[Real](k.suc) * v).cos
    sin_nt(k.suc, v) = (from_nat[Real](k.suc) * v).sin
    (from_nat[Real](k.suc) * v).sin = sin_nt(k.suc, v)
    cos_nt(k.suc, v) = (from_nat[Real](k.suc) * v).cos
    (from_nat[Real](k.suc) * v).cos = cos_nt(k.suc, v)
    sincos(from_nat[Real](k.suc) * v) = sin_nt(k.suc, v) * cos_nt(k.suc, v)
    sincos(from_nat[Real](k.suc) * u) - sincos(from_nat[Real](k.suc) * v) = sin_nt(k.suc, u) * cos_nt(k.suc, u) - sin_nt(k.suc, v) * cos_nt(k.suc, v)

    (sincos(from_nat[Real](k.suc) * u) - sincos(from_nat[Real](k.suc) * v)).abs = (sin_nt(k.suc, u) * cos_nt(k.suc, u) - sin_nt(k.suc, v) * cos_nt(k.suc, v)).abs

    (sin_nt(k.suc, u) * cos_nt(k.suc, u) - sin_nt(k.suc, v) * cos_nt(k.suc, v)).abs <= two * (from_nat[Real](k.suc) * u - from_nat[Real](k.suc) * v).abs

    (pointwise_mul(sin_nt(k.suc), cos_nt(k.suc), u) - pointwise_mul(sin_nt(k.suc), cos_nt(k.suc), v)).abs <= two * (from_nat[Real](k.suc) * u - from_nat[Real](k.suc) * v).abs

    mul_sub_distrib_left(from_nat[Real](k.suc), u, v)
    from_nat[Real](k.suc) * u - from_nat[Real](k.suc) * v = from_nat[Real](k.suc) * (u - v)
    (from_nat[Real](k.suc) * u - from_nat[Real](k.suc) * v).abs = (from_nat[Real](k.suc) * (u - v)).abs

    mul_abs(from_nat[Real](k.suc), u - v)
    (from_nat[Real](k.suc) * (u - v)).abs = (from_nat[Real](k.suc)).abs * (u - v).abs
    from_nat_abs(k.suc)
    (from_nat[Real](k.suc)).abs = from_nat[Real](k.suc)
    (from_nat[Real](k.suc) * (u - v)).abs = from_nat[Real](k.suc) * (u - v).abs
    (from_nat[Real](k.suc) * u - from_nat[Real](k.suc) * v).abs = from_nat[Real](k.suc) * (u - v).abs

    (pointwise_mul(sin_nt(k.suc), cos_nt(k.suc), u) - pointwise_mul(sin_nt(k.suc), cos_nt(k.suc), v)).abs <= two * (from_nat[Real](k.suc) * (u - v).abs)

    mul_assoc(two, from_nat[Real](k.suc), (u - v).abs)
    two * (from_nat[Real](k.suc) * (u - v).abs) = (two * from_nat[Real](k.suc)) * (u - v).abs
    (pointwise_mul(sin_nt(k.suc), cos_nt(k.suc), u) - pointwise_mul(sin_nt(k.suc), cos_nt(k.suc), v)).abs <= (two * from_nat[Real](k.suc)) * (u - v).abs

}

/// The square of the cosine harmonic is integrable on [0, 2*pi].
theorem cos_nt_sq_integrable(k: Nat) {
    is_integrable(pointwise_mul(cos_nt(k.suc), cos_nt(k.suc)), Real.0, two_pi)
} by {
        two_pi_nonneg
        Real.0 <= two_pi
        from_nat_nonneg_real(k.suc)
        Real.0 <= from_nat[Real](k.suc)
        two_positive
        two.is_positive
        pos_gt_zero(two)
        two > Real.0
        lt_imp_lte(Real.0, two)
        Real.0 <= two
        two >= Real.0
        from_nat[Real](k.suc) >= Real.0
        mul_nonneg(two, from_nat[Real](k.suc))
        two >= Real.0 and from_nat[Real](k.suc) >= Real.0 implies two * from_nat[Real](k.suc) >= Real.0
        two * from_nat[Real](k.suc) >= Real.0
        Real.0 <= two * from_nat[Real](k.suc)
        g_cos_nt_sq_continuous(k)
        continuous(g_cos_nt_sq(k.suc))
        g_cos_nt_sq_derivative(k)
        is_derivative_fn(g_cos_nt_sq(k.suc), pointwise_mul(cos_nt(k.suc), cos_nt(k.suc)))
        forall(u: Real, v: Real) {
            if interval_contains(Real.0, two_pi, u) and interval_contains(Real.0, two_pi, v) {
                cos_nt_sq_lipschitz(k, u, v)
                (pointwise_mul(cos_nt(k.suc), cos_nt(k.suc), u) - pointwise_mul(cos_nt(k.suc), cos_nt(k.suc), v)).abs <= (two * from_nat[Real](k.suc)) * (u - v).abs

            }
        }
        forall(t: Real) {
            if interval_contains(Real.0, two_pi, t) {
                cos_nt_sq_abs_le_one(k, t)
                (pointwise_mul(cos_nt(k.suc), cos_nt(k.suc), t)).abs <= Real.1
                abs_le_one_imp_lower(pointwise_mul(cos_nt(k.suc), cos_nt(k.suc), t))
                -Real.1 <= pointwise_mul(cos_nt(k.suc), cos_nt(k.suc), t)
            }
        }
        forall(t: Real) {
            if interval_contains(Real.0, two_pi, t) {
                cos_nt_sq_abs_le_one(k, t)
                (pointwise_mul(cos_nt(k.suc), cos_nt(k.suc), t)).abs <= Real.1
                abs_le_one_imp_upper(pointwise_mul(cos_nt(k.suc), cos_nt(k.suc), t))
                pointwise_mul(cos_nt(k.suc), cos_nt(k.suc), t) <= Real.1
            }
        }
        Real.0 <= two_pi
        (Real.0 <= two_pi) and Real.0 <= two * from_nat[Real](k.suc)
        ((Real.0 <= two_pi) and Real.0 <= two * from_nat[Real](k.suc)) and continuous(g_cos_nt_sq(k.suc))
        (((Real.0 <= two_pi) and Real.0 <= two * from_nat[Real](k.suc)) and continuous(g_cos_nt_sq(k.suc))) and is_derivative_fn(g_cos_nt_sq(k.suc), pointwise_mul(cos_nt(k.suc), cos_nt(k.suc)))

        ((((Real.0 <= two_pi) and Real.0 <= two * from_nat[Real](k.suc)) and continuous(g_cos_nt_sq(k.suc))) and is_derivative_fn(g_cos_nt_sq(k.suc), pointwise_mul(cos_nt(k.suc), cos_nt(k.suc)))) and
 forall(u: Real, v: Real) {

                interval_contains(Real.0, two_pi, u) and interval_contains(Real.0, two_pi, v)
                implies (pointwise_mul(cos_nt(k.suc), cos_nt(k.suc), u) - pointwise_mul(cos_nt(k.suc), cos_nt(k.suc), v)).abs <= (two * from_nat[Real](k.suc)) * (u - v).abs

            }
        (((((Real.0 <= two_pi) and Real.0 <= two * from_nat[Real](k.suc)) and continuous(g_cos_nt_sq(k.suc))) and is_derivative_fn(g_cos_nt_sq(k.suc), pointwise_mul(cos_nt(k.suc), cos_nt(k.suc)))) and
 forall(u: Real, v: Real) {

                interval_contains(Real.0, two_pi, u) and interval_contains(Real.0, two_pi, v)
                implies (pointwise_mul(cos_nt(k.suc), cos_nt(k.suc), u) - pointwise_mul(cos_nt(k.suc), cos_nt(k.suc), v)).abs <= (two * from_nat[Real](k.suc)) * (u - v).abs

            }) and forall(t: Real) {
                interval_contains(Real.0, two_pi, t) implies -Real.1 <= pointwise_mul(cos_nt(k.suc), cos_nt(k.suc), t)
            }
        ((((((Real.0 <= two_pi) and Real.0 <= two * from_nat[Real](k.suc)) and continuous(g_cos_nt_sq(k.suc))) and is_derivative_fn(g_cos_nt_sq(k.suc), pointwise_mul(cos_nt(k.suc), cos_nt(k.suc)))) and
 forall(u: Real, v: Real) {

                interval_contains(Real.0, two_pi, u) and interval_contains(Real.0, two_pi, v)
                implies (pointwise_mul(cos_nt(k.suc), cos_nt(k.suc), u) - pointwise_mul(cos_nt(k.suc), cos_nt(k.suc), v)).abs <= (two * from_nat[Real](k.suc)) * (u - v).abs

            }) and forall(t: Real) {
                interval_contains(Real.0, two_pi, t) implies -Real.1 <= pointwise_mul(cos_nt(k.suc), cos_nt(k.suc), t)
            }) and forall(t: Real) {
                interval_contains(Real.0, two_pi, t) implies pointwise_mul(cos_nt(k.suc), cos_nt(k.suc), t) <= Real.1
            }
        if ((((((Real.0 <= two_pi) and Real.0 <= two * from_nat[Real](k.suc)) and continuous(g_cos_nt_sq(k.suc))) and is_derivative_fn(g_cos_nt_sq(k.suc), pointwise_mul(cos_nt(k.suc), cos_nt(k.suc)))) and
 forall(u: Real, v: Real) {

                interval_contains(Real.0, two_pi, u) and interval_contains(Real.0, two_pi, v)
                implies (pointwise_mul(cos_nt(k.suc), cos_nt(k.suc), u) - pointwise_mul(cos_nt(k.suc), cos_nt(k.suc), v)).abs <= (two * from_nat[Real](k.suc)) * (u - v).abs

            }) and forall(t: Real) {
                interval_contains(Real.0, two_pi, t) implies -Real.1 <= pointwise_mul(cos_nt(k.suc), cos_nt(k.suc), t)
            }) and forall(t: Real) {
                interval_contains(Real.0, two_pi, t) implies pointwise_mul(cos_nt(k.suc), cos_nt(k.suc), t) <= Real.1
            } {
            fn_integrable_gen(pointwise_mul(cos_nt(k.suc), cos_nt(k.suc)), g_cos_nt_sq(k.suc), Real.0, two_pi, two * from_nat[Real](k.suc), -Real.1, Real.1)

            is_integrable(pointwise_mul(cos_nt(k.suc), cos_nt(k.suc)), Real.0, two_pi)
        }
}

/// The integral of kx.cos^2 over [0, 2*pi] is pi, for k >= 1.
theorem integral_cos_nt_sq(k: Nat) {
    integral(pointwise_mul(cos_nt(k.suc), cos_nt(k.suc)), Real.0, two_pi) = pi
} by {
        two_pi_nonneg
        Real.0 <= two_pi
        g_cos_nt_sq_continuous(k)
        continuous(g_cos_nt_sq(k.suc))
        g_cos_nt_sq_derivative(k)
        is_derivative_fn(g_cos_nt_sq(k.suc), pointwise_mul(cos_nt(k.suc), cos_nt(k.suc)))
        cos_nt_sq_integrable(k)
        is_integrable(pointwise_mul(cos_nt(k.suc), cos_nt(k.suc)), Real.0, two_pi)
        forall(t: Real) {
            if interval_contains(Real.0, two_pi, t) {
                cos_nt_sq_abs_le_one(k, t)
                (pointwise_mul(cos_nt(k.suc), cos_nt(k.suc), t)).abs <= Real.1
                abs_le_one_imp_lower(pointwise_mul(cos_nt(k.suc), cos_nt(k.suc), t))
                -Real.1 <= pointwise_mul(cos_nt(k.suc), cos_nt(k.suc), t)
            }
        }
        forall(t: Real) {
            if interval_contains(Real.0, two_pi, t) {
                cos_nt_sq_abs_le_one(k, t)
                (pointwise_mul(cos_nt(k.suc), cos_nt(k.suc), t)).abs <= Real.1
                abs_le_one_imp_upper(pointwise_mul(cos_nt(k.suc), cos_nt(k.suc), t))
                pointwise_mul(cos_nt(k.suc), cos_nt(k.suc), t) <= Real.1
            }
        }
        Real.0 <= two_pi
        (Real.0 <= two_pi) and continuous(g_cos_nt_sq(k.suc))
        ((Real.0 <= two_pi) and continuous(g_cos_nt_sq(k.suc))) and is_derivative_fn(g_cos_nt_sq(k.suc), pointwise_mul(cos_nt(k.suc), cos_nt(k.suc)))

        (((Real.0 <= two_pi) and continuous(g_cos_nt_sq(k.suc))) and is_derivative_fn(g_cos_nt_sq(k.suc), pointwise_mul(cos_nt(k.suc), cos_nt(k.suc)))) and
 is_integrable(pointwise_mul(cos_nt(k.suc), cos_nt(k.suc)), Real.0, two_pi)

        ((((Real.0 <= two_pi) and continuous(g_cos_nt_sq(k.suc))) and is_derivative_fn(g_cos_nt_sq(k.suc), pointwise_mul(cos_nt(k.suc), cos_nt(k.suc)))) and
 is_integrable(pointwise_mul(cos_nt(k.suc), cos_nt(k.suc)), Real.0, two_pi)) and forall(t: Real) {

                interval_contains(Real.0, two_pi, t) implies -Real.1 <= pointwise_mul(cos_nt(k.suc), cos_nt(k.suc), t)
            }
        (((((Real.0 <= two_pi) and continuous(g_cos_nt_sq(k.suc))) and is_derivative_fn(g_cos_nt_sq(k.suc), pointwise_mul(cos_nt(k.suc), cos_nt(k.suc)))) and
 is_integrable(pointwise_mul(cos_nt(k.suc), cos_nt(k.suc)), Real.0, two_pi)) and forall(t: Real) {

                interval_contains(Real.0, two_pi, t) implies -Real.1 <= pointwise_mul(cos_nt(k.suc), cos_nt(k.suc), t)
            }) and forall(t: Real) {
                interval_contains(Real.0, two_pi, t) implies pointwise_mul(cos_nt(k.suc), cos_nt(k.suc), t) <= Real.1
            }
        if (((((Real.0 <= two_pi) and continuous(g_cos_nt_sq(k.suc))) and is_derivative_fn(g_cos_nt_sq(k.suc), pointwise_mul(cos_nt(k.suc), cos_nt(k.suc)))) and
 is_integrable(pointwise_mul(cos_nt(k.suc), cos_nt(k.suc)), Real.0, two_pi)) and forall(t: Real) {

                interval_contains(Real.0, two_pi, t) implies -Real.1 <= pointwise_mul(cos_nt(k.suc), cos_nt(k.suc), t)
            }) and forall(t: Real) {
                interval_contains(Real.0, two_pi, t) implies pointwise_mul(cos_nt(k.suc), cos_nt(k.suc), t) <= Real.1
            } {
            ftc2_general(pointwise_mul(cos_nt(k.suc), cos_nt(k.suc)), g_cos_nt_sq(k.suc), Real.0, two_pi, -Real.1, Real.1)

            integral(pointwise_mul(cos_nt(k.suc), cos_nt(k.suc)), Real.0, two_pi) = g_cos_nt_sq(k.suc)(two_pi) - g_cos_nt_sq(k.suc)(Real.0)

            pointwise_add_apply(pointwise_mul(constant[Real, Real](Real.one_half), identity_fn[Real]), pointwise_mul(constant[Real, Real](Real.one_half / (two * from_nat[Real](k.suc))),
 sin_nt(Nat.2 * k.suc)), two_pi)

            pointwise_add(pointwise_mul(constant[Real, Real](Real.one_half), identity_fn[Real]), pointwise_mul(constant[Real, Real](Real.one_half / (two * from_nat[Real](k.suc))),
 sin_nt(Nat.2 * k.suc)), two_pi) =
 pointwise_mul(constant[Real, Real](Real.one_half), identity_fn[Real], two_pi) +

                    pointwise_mul(constant[Real, Real](Real.one_half / (two * from_nat[Real](k.suc))), sin_nt(Nat.2 * k.suc), two_pi)

            pointwise_mul_apply(constant[Real, Real](Real.one_half), identity_fn[Real], two_pi)
            pointwise_mul(constant[Real, Real](Real.one_half), identity_fn[Real], two_pi) = constant[Real, Real](Real.one_half, two_pi) * identity_fn[Real](two_pi)

            constant[Real, Real](Real.one_half, two_pi) = Real.one_half
            identity_fn[Real](two_pi) = two_pi
            pointwise_mul(constant[Real, Real](Real.one_half), identity_fn[Real], two_pi) = Real.one_half * two_pi

            two_pi = pi + pi
            pi + pi = two * pi
            add_self_two(pi)
            two_pi = two * pi
            Real.one_half * two_pi = Real.one_half * (two * pi)
            Real.one_half = Real.1 / two
            Real.one_half * (two * pi) = (Real.1 / two) * (two * pi)
            half_twice(pi)
            (Real.1 / two) * (two * pi) = pi
            Real.one_half * (two * pi) = pi
            Real.one_half * two_pi = pi
            pointwise_mul(constant[Real, Real](Real.one_half), identity_fn[Real], two_pi) = pi
            pointwise_mul_apply(constant[Real, Real](Real.one_half / (two * from_nat[Real](k.suc))), sin_nt(Nat.2 * k.suc), two_pi)

            pointwise_mul(constant[Real, Real](Real.one_half / (two * from_nat[Real](k.suc))), sin_nt(Nat.2 * k.suc), two_pi) =
 constant[Real, Real](Real.one_half / (two * from_nat[Real](k.suc)), two_pi) *

                    sin_nt(Nat.2 * k.suc, two_pi)
            constant[Real, Real](Real.one_half / (two * from_nat[Real](k.suc)), two_pi) = Real.one_half / (two * from_nat[Real](k.suc))

            pointwise_mul(constant[Real, Real](Real.one_half / (two * from_nat[Real](k.suc))), sin_nt(Nat.2 * k.suc), two_pi) =
 (Real.one_half / (two * from_nat[Real](k.suc))) * sin_nt(Nat.2 * k.suc, two_pi)

            sin_nt_two_pi_zero(Nat.2 * k.suc)
            sin_nt(Nat.2 * k.suc, two_pi) = Real.0
            (Real.one_half / (two * from_nat[Real](k.suc))) * sin_nt(Nat.2 * k.suc, two_pi) = Real.0
            pointwise_mul(constant[Real, Real](Real.one_half / (two * from_nat[Real](k.suc))), sin_nt(Nat.2 * k.suc), two_pi) = Real.0

            pointwise_add(pointwise_mul(constant[Real, Real](Real.one_half), identity_fn[Real]), pointwise_mul(constant[Real, Real](Real.one_half / (two * from_nat[Real](k.suc))),
 sin_nt(Nat.2 * k.suc)), two_pi) = pi

            g_cos_nt_sq(k.suc)(two_pi) = pi
            pointwise_add_apply(pointwise_mul(constant[Real, Real](Real.one_half), identity_fn[Real]), pointwise_mul(constant[Real, Real](Real.one_half / (two * from_nat[Real](k.suc))),
 sin_nt(Nat.2 * k.suc)), Real.0)

            pointwise_add(pointwise_mul(constant[Real, Real](Real.one_half), identity_fn[Real]), pointwise_mul(constant[Real, Real](Real.one_half / (two * from_nat[Real](k.suc))),
 sin_nt(Nat.2 * k.suc)), Real.0) =
 pointwise_mul(constant[Real, Real](Real.one_half), identity_fn[Real], Real.0) +

                    pointwise_mul(constant[Real, Real](Real.one_half / (two * from_nat[Real](k.suc))), sin_nt(Nat.2 * k.suc), Real.0)

            pointwise_mul_apply(constant[Real, Real](Real.one_half), identity_fn[Real], Real.0)
            pointwise_mul(constant[Real, Real](Real.one_half), identity_fn[Real], Real.0) = constant[Real, Real](Real.one_half, Real.0) * identity_fn[Real](Real.0)

            constant[Real, Real](Real.one_half, Real.0) = Real.one_half
            identity_fn[Real](Real.0) = Real.0
            pointwise_mul(constant[Real, Real](Real.one_half), identity_fn[Real], Real.0) = Real.one_half * Real.0

            Real.one_half * Real.0 = Real.0
            pointwise_mul_apply(constant[Real, Real](Real.one_half / (two * from_nat[Real](k.suc))), sin_nt(Nat.2 * k.suc), Real.0)

            pointwise_mul(constant[Real, Real](Real.one_half / (two * from_nat[Real](k.suc))), sin_nt(Nat.2 * k.suc), Real.0) =
 constant[Real, Real](Real.one_half / (two * from_nat[Real](k.suc)), Real.0) *

                    sin_nt(Nat.2 * k.suc, Real.0)
            constant[Real, Real](Real.one_half / (two * from_nat[Real](k.suc)), Real.0) = Real.one_half / (two * from_nat[Real](k.suc))

            pointwise_mul(constant[Real, Real](Real.one_half / (two * from_nat[Real](k.suc))), sin_nt(Nat.2 * k.suc), Real.0) =
 (Real.one_half / (two * from_nat[Real](k.suc))) * sin_nt(Nat.2 * k.suc, Real.0)

            sin_nt_at_zero(Nat.2 * k.suc)
            sin_nt(Nat.2 * k.suc, Real.0) = Real.0
            (Real.one_half / (two * from_nat[Real](k.suc))) * sin_nt(Nat.2 * k.suc, Real.0) = Real.0
            pointwise_mul(constant[Real, Real](Real.one_half / (two * from_nat[Real](k.suc))), sin_nt(Nat.2 * k.suc), Real.0) = Real.0

            pointwise_add(pointwise_mul(constant[Real, Real](Real.one_half), identity_fn[Real]), pointwise_mul(constant[Real, Real](Real.one_half / (two * from_nat[Real](k.suc))),
 sin_nt(Nat.2 * k.suc)), Real.0) = Real.0

            g_cos_nt_sq(k.suc)(Real.0) = Real.0
            g_cos_nt_sq(k.suc)(two_pi) - g_cos_nt_sq(k.suc)(Real.0) = pi - Real.0
            pi - Real.0 = pi
            integral(pointwise_mul(cos_nt(k.suc), cos_nt(k.suc)), Real.0, two_pi) = pi
        }
}

/// The product (kx).sincos(kx) is integrable on [0, 2*pi].
theorem sin_nt_cos_nt_integrable(k: Nat) {
    is_integrable(pointwise_mul(sin_nt(k.suc), cos_nt(k.suc)), Real.0, two_pi)
} by {
        two_pi_nonneg
        Real.0 <= two_pi
        from_nat_nonneg_real(k.suc)
        Real.0 <= from_nat[Real](k.suc)
        two_positive
        two.is_positive
        pos_gt_zero(two)
        two > Real.0
        lt_imp_lte(Real.0, two)
        Real.0 <= two
        two >= Real.0
        from_nat[Real](k.suc) >= Real.0
        mul_nonneg(two, from_nat[Real](k.suc))
        two >= Real.0 and from_nat[Real](k.suc) >= Real.0 implies two * from_nat[Real](k.suc) >= Real.0
        two * from_nat[Real](k.suc) >= Real.0
        Real.0 <= two * from_nat[Real](k.suc)
        g_sin_cos_continuous(k)
        continuous(g_sin_cos(k.suc))
        g_sin_cos_derivative(k)
        is_derivative_fn(g_sin_cos(k.suc), pointwise_mul(sin_nt(k.suc), cos_nt(k.suc)))
        forall(u: Real, v: Real) {
            if interval_contains(Real.0, two_pi, u) and interval_contains(Real.0, two_pi, v) {
                sin_nt_cos_nt_lipschitz(k, u, v)
                (pointwise_mul(sin_nt(k.suc), cos_nt(k.suc), u) - pointwise_mul(sin_nt(k.suc), cos_nt(k.suc), v)).abs <= (two * from_nat[Real](k.suc)) * (u - v).abs

            }
        }
        forall(t: Real) {
            if interval_contains(Real.0, two_pi, t) {
                sin_nt_cos_nt_abs_le_one(k, t)
                (pointwise_mul(sin_nt(k.suc), cos_nt(k.suc), t)).abs <= Real.1
                abs_le_one_imp_lower(pointwise_mul(sin_nt(k.suc), cos_nt(k.suc), t))
                -Real.1 <= pointwise_mul(sin_nt(k.suc), cos_nt(k.suc), t)
            }
        }
        forall(t: Real) {
            if interval_contains(Real.0, two_pi, t) {
                sin_nt_cos_nt_abs_le_one(k, t)
                (pointwise_mul(sin_nt(k.suc), cos_nt(k.suc), t)).abs <= Real.1
                abs_le_one_imp_upper(pointwise_mul(sin_nt(k.suc), cos_nt(k.suc), t))
                pointwise_mul(sin_nt(k.suc), cos_nt(k.suc), t) <= Real.1
            }
        }
        Real.0 <= two_pi
        (Real.0 <= two_pi) and Real.0 <= two * from_nat[Real](k.suc)
        ((Real.0 <= two_pi) and Real.0 <= two * from_nat[Real](k.suc)) and continuous(g_sin_cos(k.suc))
        (((Real.0 <= two_pi) and Real.0 <= two * from_nat[Real](k.suc)) and continuous(g_sin_cos(k.suc))) and is_derivative_fn(g_sin_cos(k.suc), pointwise_mul(sin_nt(k.suc), cos_nt(k.suc)))

        ((((Real.0 <= two_pi) and Real.0 <= two * from_nat[Real](k.suc)) and continuous(g_sin_cos(k.suc))) and is_derivative_fn(g_sin_cos(k.suc), pointwise_mul(sin_nt(k.suc), cos_nt(k.suc)))) and
 forall(u: Real, v: Real) {

                interval_contains(Real.0, two_pi, u) and interval_contains(Real.0, two_pi, v)
                implies (pointwise_mul(sin_nt(k.suc), cos_nt(k.suc), u) - pointwise_mul(sin_nt(k.suc), cos_nt(k.suc), v)).abs <= (two * from_nat[Real](k.suc)) * (u - v).abs

            }
        (((((Real.0 <= two_pi) and Real.0 <= two * from_nat[Real](k.suc)) and continuous(g_sin_cos(k.suc))) and is_derivative_fn(g_sin_cos(k.suc), pointwise_mul(sin_nt(k.suc), cos_nt(k.suc)))) and
 forall(u: Real, v: Real) {

                interval_contains(Real.0, two_pi, u) and interval_contains(Real.0, two_pi, v)
                implies (pointwise_mul(sin_nt(k.suc), cos_nt(k.suc), u) - pointwise_mul(sin_nt(k.suc), cos_nt(k.suc), v)).abs <= (two * from_nat[Real](k.suc)) * (u - v).abs

            }) and forall(t: Real) {
                interval_contains(Real.0, two_pi, t) implies -Real.1 <= pointwise_mul(sin_nt(k.suc), cos_nt(k.suc), t)
            }
        ((((((Real.0 <= two_pi) and Real.0 <= two * from_nat[Real](k.suc)) and continuous(g_sin_cos(k.suc))) and is_derivative_fn(g_sin_cos(k.suc), pointwise_mul(sin_nt(k.suc), cos_nt(k.suc)))) and
 forall(u: Real, v: Real) {

                interval_contains(Real.0, two_pi, u) and interval_contains(Real.0, two_pi, v)
                implies (pointwise_mul(sin_nt(k.suc), cos_nt(k.suc), u) - pointwise_mul(sin_nt(k.suc), cos_nt(k.suc), v)).abs <= (two * from_nat[Real](k.suc)) * (u - v).abs

            }) and forall(t: Real) {
                interval_contains(Real.0, two_pi, t) implies -Real.1 <= pointwise_mul(sin_nt(k.suc), cos_nt(k.suc), t)
            }) and forall(t: Real) {
                interval_contains(Real.0, two_pi, t) implies pointwise_mul(sin_nt(k.suc), cos_nt(k.suc), t) <= Real.1
            }
        if ((((((Real.0 <= two_pi) and Real.0 <= two * from_nat[Real](k.suc)) and continuous(g_sin_cos(k.suc))) and is_derivative_fn(g_sin_cos(k.suc), pointwise_mul(sin_nt(k.suc), cos_nt(k.suc)))) and
 forall(u: Real, v: Real) {

                interval_contains(Real.0, two_pi, u) and interval_contains(Real.0, two_pi, v)
                implies (pointwise_mul(sin_nt(k.suc), cos_nt(k.suc), u) - pointwise_mul(sin_nt(k.suc), cos_nt(k.suc), v)).abs <= (two * from_nat[Real](k.suc)) * (u - v).abs

            }) and forall(t: Real) {
                interval_contains(Real.0, two_pi, t) implies -Real.1 <= pointwise_mul(sin_nt(k.suc), cos_nt(k.suc), t)
            }) and forall(t: Real) {
                interval_contains(Real.0, two_pi, t) implies pointwise_mul(sin_nt(k.suc), cos_nt(k.suc), t) <= Real.1
            } {
            fn_integrable_gen(pointwise_mul(sin_nt(k.suc), cos_nt(k.suc)), g_sin_cos(k.suc), Real.0, two_pi, two * from_nat[Real](k.suc), -Real.1, Real.1)

            is_integrable(pointwise_mul(sin_nt(k.suc), cos_nt(k.suc)), Real.0, two_pi)
        }
}

/// The integral of (kx).sincos(kx) over [0, 2*pi] is zero, for k >= 1.
theorem integral_sin_nt_cos_nt(k: Nat) {
    integral(pointwise_mul(sin_nt(k.suc), cos_nt(k.suc)), Real.0, two_pi) = Real.0
} by {
        two_pi_nonneg
        Real.0 <= two_pi
        g_sin_cos_continuous(k)
        continuous(g_sin_cos(k.suc))
        g_sin_cos_derivative(k)
        is_derivative_fn(g_sin_cos(k.suc), pointwise_mul(sin_nt(k.suc), cos_nt(k.suc)))
        sin_nt_cos_nt_integrable(k)
        is_integrable(pointwise_mul(sin_nt(k.suc), cos_nt(k.suc)), Real.0, two_pi)
        forall(t: Real) {
            if interval_contains(Real.0, two_pi, t) {
                sin_nt_cos_nt_abs_le_one(k, t)
                (pointwise_mul(sin_nt(k.suc), cos_nt(k.suc), t)).abs <= Real.1
                abs_le_one_imp_lower(pointwise_mul(sin_nt(k.suc), cos_nt(k.suc), t))
                -Real.1 <= pointwise_mul(sin_nt(k.suc), cos_nt(k.suc), t)
            }
        }
        forall(t: Real) {
            if interval_contains(Real.0, two_pi, t) {
                sin_nt_cos_nt_abs_le_one(k, t)
                (pointwise_mul(sin_nt(k.suc), cos_nt(k.suc), t)).abs <= Real.1
                abs_le_one_imp_upper(pointwise_mul(sin_nt(k.suc), cos_nt(k.suc), t))
                pointwise_mul(sin_nt(k.suc), cos_nt(k.suc), t) <= Real.1
            }
        }
        Real.0 <= two_pi
        (Real.0 <= two_pi) and continuous(g_sin_cos(k.suc))
        ((Real.0 <= two_pi) and continuous(g_sin_cos(k.suc))) and is_derivative_fn(g_sin_cos(k.suc), pointwise_mul(sin_nt(k.suc), cos_nt(k.suc)))

        (((Real.0 <= two_pi) and continuous(g_sin_cos(k.suc))) and is_derivative_fn(g_sin_cos(k.suc), pointwise_mul(sin_nt(k.suc), cos_nt(k.suc)))) and
 is_integrable(pointwise_mul(sin_nt(k.suc), cos_nt(k.suc)), Real.0, two_pi)

        ((((Real.0 <= two_pi) and continuous(g_sin_cos(k.suc))) and is_derivative_fn(g_sin_cos(k.suc), pointwise_mul(sin_nt(k.suc), cos_nt(k.suc)))) and
 is_integrable(pointwise_mul(sin_nt(k.suc), cos_nt(k.suc)), Real.0, two_pi)) and forall(t: Real) {

                interval_contains(Real.0, two_pi, t) implies -Real.1 <= pointwise_mul(sin_nt(k.suc), cos_nt(k.suc), t)
            }
        (((((Real.0 <= two_pi) and continuous(g_sin_cos(k.suc))) and is_derivative_fn(g_sin_cos(k.suc), pointwise_mul(sin_nt(k.suc), cos_nt(k.suc)))) and
 is_integrable(pointwise_mul(sin_nt(k.suc), cos_nt(k.suc)), Real.0, two_pi)) and forall(t: Real) {

                interval_contains(Real.0, two_pi, t) implies -Real.1 <= pointwise_mul(sin_nt(k.suc), cos_nt(k.suc), t)
            }) and forall(t: Real) {
                interval_contains(Real.0, two_pi, t) implies pointwise_mul(sin_nt(k.suc), cos_nt(k.suc), t) <= Real.1
            }
        if (((((Real.0 <= two_pi) and continuous(g_sin_cos(k.suc))) and is_derivative_fn(g_sin_cos(k.suc), pointwise_mul(sin_nt(k.suc), cos_nt(k.suc)))) and
 is_integrable(pointwise_mul(sin_nt(k.suc), cos_nt(k.suc)), Real.0, two_pi)) and forall(t: Real) {

                interval_contains(Real.0, two_pi, t) implies -Real.1 <= pointwise_mul(sin_nt(k.suc), cos_nt(k.suc), t)
            }) and forall(t: Real) {
                interval_contains(Real.0, two_pi, t) implies pointwise_mul(sin_nt(k.suc), cos_nt(k.suc), t) <= Real.1
            } {
            ftc2_general(pointwise_mul(sin_nt(k.suc), cos_nt(k.suc)), g_sin_cos(k.suc), Real.0, two_pi, -Real.1, Real.1)

            integral(pointwise_mul(sin_nt(k.suc), cos_nt(k.suc)), Real.0, two_pi) = g_sin_cos(k.suc)(two_pi) - g_sin_cos(k.suc)(Real.0)

            pointwise_neg_apply(pointwise_mul(constant[Real, Real](Real.one_half / (two * from_nat[Real](k.suc))), cos_nt(Nat.2 * k.suc)), two_pi)

            pointwise_neg(pointwise_mul(constant[Real, Real](Real.one_half / (two * from_nat[Real](k.suc))), cos_nt(Nat.2 * k.suc)), two_pi) =
 -(pointwise_mul(constant[Real, Real](Real.one_half / (two * from_nat[Real](k.suc))),
 cos_nt(Nat.2 * k.suc), two_pi))

            pointwise_mul_apply(constant[Real, Real](Real.one_half / (two * from_nat[Real](k.suc))), cos_nt(Nat.2 * k.suc), two_pi)

            pointwise_mul(constant[Real, Real](Real.one_half / (two * from_nat[Real](k.suc))), cos_nt(Nat.2 * k.suc), two_pi) =
 constant[Real, Real](Real.one_half / (two * from_nat[Real](k.suc)), two_pi) *

                    cos_nt(Nat.2 * k.suc, two_pi)
            constant[Real, Real](Real.one_half / (two * from_nat[Real](k.suc)), two_pi) = Real.one_half / (two * from_nat[Real](k.suc))

            pointwise_mul(constant[Real, Real](Real.one_half / (two * from_nat[Real](k.suc))), cos_nt(Nat.2 * k.suc), two_pi) =
 (Real.one_half / (two * from_nat[Real](k.suc))) * cos_nt(Nat.2 * k.suc, two_pi)

            cos_nt_two_pi_one(Nat.2 * k.suc)
            cos_nt(Nat.2 * k.suc, two_pi) = Real.1
            (Real.one_half / (two * from_nat[Real](k.suc))) * cos_nt(Nat.2 * k.suc, two_pi) = Real.one_half / (two * from_nat[Real](k.suc))

            pointwise_mul(constant[Real, Real](Real.one_half / (two * from_nat[Real](k.suc))), cos_nt(Nat.2 * k.suc), two_pi) =
 Real.one_half / (two * from_nat[Real](k.suc))

            pointwise_neg(pointwise_mul(constant[Real, Real](Real.one_half / (two * from_nat[Real](k.suc))), cos_nt(Nat.2 * k.suc)), two_pi) =
 -(Real.one_half / (two * from_nat[Real](k.suc)))

            g_sin_cos(k.suc)(two_pi) = -(Real.one_half / (two * from_nat[Real](k.suc)))
            pointwise_neg_apply(pointwise_mul(constant[Real, Real](Real.one_half / (two * from_nat[Real](k.suc))), cos_nt(Nat.2 * k.suc)), Real.0)

            pointwise_neg(pointwise_mul(constant[Real, Real](Real.one_half / (two * from_nat[Real](k.suc))), cos_nt(Nat.2 * k.suc)), Real.0) =
 -(pointwise_mul(constant[Real, Real](Real.one_half / (two * from_nat[Real](k.suc))),
 cos_nt(Nat.2 * k.suc), Real.0))

            pointwise_mul_apply(constant[Real, Real](Real.one_half / (two * from_nat[Real](k.suc))), cos_nt(Nat.2 * k.suc), Real.0)

            pointwise_mul(constant[Real, Real](Real.one_half / (two * from_nat[Real](k.suc))), cos_nt(Nat.2 * k.suc), Real.0) =
 constant[Real, Real](Real.one_half / (two * from_nat[Real](k.suc)), Real.0) *

                    cos_nt(Nat.2 * k.suc, Real.0)
            constant[Real, Real](Real.one_half / (two * from_nat[Real](k.suc)), Real.0) = Real.one_half / (two * from_nat[Real](k.suc))

            pointwise_mul(constant[Real, Real](Real.one_half / (two * from_nat[Real](k.suc))), cos_nt(Nat.2 * k.suc), Real.0) =
 (Real.one_half / (two * from_nat[Real](k.suc))) * cos_nt(Nat.2 * k.suc, Real.0)

            cos_nt_at_zero(Nat.2 * k.suc)
            cos_nt(Nat.2 * k.suc, Real.0) = Real.1
            (Real.one_half / (two * from_nat[Real](k.suc))) * cos_nt(Nat.2 * k.suc, Real.0) = Real.one_half / (two * from_nat[Real](k.suc))

            pointwise_mul(constant[Real, Real](Real.one_half / (two * from_nat[Real](k.suc))), cos_nt(Nat.2 * k.suc), Real.0) =
 Real.one_half / (two * from_nat[Real](k.suc))

            pointwise_neg(pointwise_mul(constant[Real, Real](Real.one_half / (two * from_nat[Real](k.suc))), cos_nt(Nat.2 * k.suc)), Real.0) =
 -(Real.one_half / (two * from_nat[Real](k.suc)))

            g_sin_cos(k.suc)(Real.0) = -(Real.one_half / (two * from_nat[Real](k.suc)))
            g_sin_cos(k.suc)(two_pi) - g_sin_cos(k.suc)(Real.0) = -(Real.one_half / (two * from_nat[Real](k.suc))) -

                    (-(Real.one_half / (two * from_nat[Real](k.suc))))
            -(Real.one_half / (two * from_nat[Real](k.suc))) - (-(Real.one_half / (two * from_nat[Real](k.suc)))) = Real.0
            integral(pointwise_mul(sin_nt(k.suc), cos_nt(k.suc)), Real.0, two_pi) = Real.0
        }
}

// ---------------------------------------------------------------------------
// The Fourier coefficients of the harmonics themselves, for general k
// ---------------------------------------------------------------------------

/// The b_k coefficient of f(x) = kx.sin is one.
theorem fourier_sin_coeff_sin_k(k: Nat) {
    fourier_sin_coeff(sin_nt(k.suc), k.suc) = Real.1
} by {
    forall(x: Real) {
        fourier_sin_integrand(sin_nt(k.suc), k.suc, x) = sin_nt(k.suc, x) * sin_nt(k.suc, x)
        pointwise_mul_apply(sin_nt(k.suc), sin_nt(k.suc), x)
        pointwise_mul(sin_nt(k.suc), sin_nt(k.suc), x) = sin_nt(k.suc, x) * sin_nt(k.suc, x)
        fourier_sin_integrand(sin_nt(k.suc), k.suc, x) = pointwise_mul(sin_nt(k.suc), sin_nt(k.suc), x)
    }
    function_extensionality(fourier_sin_integrand(sin_nt(k.suc), k.suc),
        pointwise_mul(sin_nt(k.suc), sin_nt(k.suc)))
    fourier_sin_integrand(sin_nt(k.suc), k.suc) = pointwise_mul(sin_nt(k.suc), sin_nt(k.suc))
    integral(fourier_sin_integrand(sin_nt(k.suc), k.suc), Real.0, two_pi) =
        integral(pointwise_mul(sin_nt(k.suc), sin_nt(k.suc)), Real.0, two_pi)
    integral_sin_nt_sq(k)
    integral(pointwise_mul(sin_nt(k.suc), sin_nt(k.suc)), Real.0, two_pi) = pi
    integral(fourier_sin_integrand(sin_nt(k.suc), k.suc), Real.0, two_pi) = pi
    fourier_sin_coeff(sin_nt(k.suc), k.suc) =
        integral(fourier_sin_integrand(sin_nt(k.suc), k.suc), Real.0, two_pi) / pi
    fourier_sin_coeff(sin_nt(k.suc), k.suc) = pi / pi
    pi_pos
    pi > Real.0
    lt_imp_ne(Real.0, pi)
    Real.0 < pi implies Real.0 != pi
    Real.0 != pi
    pi != Real.0
    div_mul_cancel_left(pi, Real.1)
    pi != Real.0 implies (pi * Real.1) / pi = Real.1
    (pi * Real.1) / pi = Real.1
    pi * Real.1 = pi
    pi / pi = Real.1
    fourier_sin_coeff(sin_nt(k.suc), k.suc) = Real.1
}

/// The a_k coefficient of f(x) = kx.sin is zero.
theorem fourier_cos_coeff_sin_k(k: Nat) {
    fourier_cos_coeff(sin_nt(k.suc), k.suc) = Real.0
} by {
    forall(x: Real) {
        fourier_cos_integrand(sin_nt(k.suc), k.suc, x) = sin_nt(k.suc, x) * cos_nt(k.suc, x)
        pointwise_mul_apply(sin_nt(k.suc), cos_nt(k.suc), x)
        pointwise_mul(sin_nt(k.suc), cos_nt(k.suc), x) = sin_nt(k.suc, x) * cos_nt(k.suc, x)
        fourier_cos_integrand(sin_nt(k.suc), k.suc, x) = pointwise_mul(sin_nt(k.suc), cos_nt(k.suc), x)
    }
    function_extensionality(fourier_cos_integrand(sin_nt(k.suc), k.suc),
        pointwise_mul(sin_nt(k.suc), cos_nt(k.suc)))
    fourier_cos_integrand(sin_nt(k.suc), k.suc) = pointwise_mul(sin_nt(k.suc), cos_nt(k.suc))
    integral(fourier_cos_integrand(sin_nt(k.suc), k.suc), Real.0, two_pi) =
        integral(pointwise_mul(sin_nt(k.suc), cos_nt(k.suc)), Real.0, two_pi)
    integral_sin_nt_cos_nt(k)
    integral(pointwise_mul(sin_nt(k.suc), cos_nt(k.suc)), Real.0, two_pi) = Real.0
    integral(fourier_cos_integrand(sin_nt(k.suc), k.suc), Real.0, two_pi) = Real.0
    fourier_cos_coeff(sin_nt(k.suc), k.suc) =
        integral(fourier_cos_integrand(sin_nt(k.suc), k.suc), Real.0, two_pi) / pi
    fourier_cos_coeff(sin_nt(k.suc), k.suc) = Real.0 / pi
    Real.0 / pi = Real.0
    fourier_cos_coeff(sin_nt(k.suc), k.suc) = Real.0
}

/// The a_k coefficient of f(x) = kx.cos is one.
theorem fourier_cos_coeff_cos_k(k: Nat) {
    fourier_cos_coeff(cos_nt(k.suc), k.suc) = Real.1
} by {
    forall(x: Real) {
        fourier_cos_integrand(cos_nt(k.suc), k.suc, x) = cos_nt(k.suc, x) * cos_nt(k.suc, x)
        pointwise_mul_apply(cos_nt(k.suc), cos_nt(k.suc), x)
        pointwise_mul(cos_nt(k.suc), cos_nt(k.suc), x) = cos_nt(k.suc, x) * cos_nt(k.suc, x)
        fourier_cos_integrand(cos_nt(k.suc), k.suc, x) = pointwise_mul(cos_nt(k.suc), cos_nt(k.suc), x)
    }
    function_extensionality(fourier_cos_integrand(cos_nt(k.suc), k.suc),
        pointwise_mul(cos_nt(k.suc), cos_nt(k.suc)))
    fourier_cos_integrand(cos_nt(k.suc), k.suc) = pointwise_mul(cos_nt(k.suc), cos_nt(k.suc))
    integral(fourier_cos_integrand(cos_nt(k.suc), k.suc), Real.0, two_pi) =
        integral(pointwise_mul(cos_nt(k.suc), cos_nt(k.suc)), Real.0, two_pi)
    integral_cos_nt_sq(k)
    integral(pointwise_mul(cos_nt(k.suc), cos_nt(k.suc)), Real.0, two_pi) = pi
    integral(fourier_cos_integrand(cos_nt(k.suc), k.suc), Real.0, two_pi) = pi
    fourier_cos_coeff(cos_nt(k.suc), k.suc) =
        integral(fourier_cos_integrand(cos_nt(k.suc), k.suc), Real.0, two_pi) / pi
    fourier_cos_coeff(cos_nt(k.suc), k.suc) = pi / pi
    pi_pos
    pi > Real.0
    lt_imp_ne(Real.0, pi)
    Real.0 < pi implies Real.0 != pi
    Real.0 != pi
    pi != Real.0
    div_mul_cancel_left(pi, Real.1)
    pi != Real.0 implies (pi * Real.1) / pi = Real.1
    (pi * Real.1) / pi = Real.1
    pi * Real.1 = pi
    pi / pi = Real.1
    fourier_cos_coeff(cos_nt(k.suc), k.suc) = Real.1
}

/// The b_k coefficient of f(x) = kx.cos is zero.
theorem fourier_sin_coeff_cos_k(k: Nat) {
    fourier_sin_coeff(cos_nt(k.suc), k.suc) = Real.0
} by {
    forall(x: Real) {
        fourier_sin_integrand(cos_nt(k.suc), k.suc, x) = cos_nt(k.suc, x) * sin_nt(k.suc, x)
        pointwise_mul_apply(sin_nt(k.suc), cos_nt(k.suc), x)
        pointwise_mul(sin_nt(k.suc), cos_nt(k.suc), x) = sin_nt(k.suc, x) * cos_nt(k.suc, x)
        cos_nt(k.suc, x) * sin_nt(k.suc, x) = sin_nt(k.suc, x) * cos_nt(k.suc, x)
        fourier_sin_integrand(cos_nt(k.suc), k.suc, x) = pointwise_mul(sin_nt(k.suc), cos_nt(k.suc), x)
    }
    function_extensionality(fourier_sin_integrand(cos_nt(k.suc), k.suc),
        pointwise_mul(sin_nt(k.suc), cos_nt(k.suc)))
    fourier_sin_integrand(cos_nt(k.suc), k.suc) = pointwise_mul(sin_nt(k.suc), cos_nt(k.suc))
    integral(fourier_sin_integrand(cos_nt(k.suc), k.suc), Real.0, two_pi) =
        integral(pointwise_mul(sin_nt(k.suc), cos_nt(k.suc)), Real.0, two_pi)
    integral_sin_nt_cos_nt(k)
    integral(pointwise_mul(sin_nt(k.suc), cos_nt(k.suc)), Real.0, two_pi) = Real.0
    integral(fourier_sin_integrand(cos_nt(k.suc), k.suc), Real.0, two_pi) = Real.0
    fourier_sin_coeff(cos_nt(k.suc), k.suc) =
        integral(fourier_sin_integrand(cos_nt(k.suc), k.suc), Real.0, two_pi) / pi
    fourier_sin_coeff(cos_nt(k.suc), k.suc) = Real.0 / pi
    Real.0 / pi = Real.0
    fourier_sin_coeff(cos_nt(k.suc), k.suc) = Real.0
}

// ---------------------------------------------------------------------------
// Parseval's identity for the general harmonics
// ---------------------------------------------------------------------------

/// Parseval for the k-th sine harmonic: (1/2*pi) integral(kx.sin^2) = 1/2.
theorem parseval_sin_k(k: Nat) {
    mean_integral(pointwise_mul(sin_nt(k.suc), sin_nt(k.suc))) = Real.one_half
} by {
    mean_integral(pointwise_mul(sin_nt(k.suc), sin_nt(k.suc))) =
        integral(pointwise_mul(sin_nt(k.suc), sin_nt(k.suc)), Real.0, two_pi) / two_pi
    integral_sin_nt_sq(k)
    integral(pointwise_mul(sin_nt(k.suc), sin_nt(k.suc)), Real.0, two_pi) = pi
    mean_integral(pointwise_mul(sin_nt(k.suc), sin_nt(k.suc))) = pi / two_pi
    pi_div_two_pi
    pi / two_pi = Real.one_half
    mean_integral(pointwise_mul(sin_nt(k.suc), sin_nt(k.suc))) = Real.one_half
}

/// Parseval for the k-th cosine harmonic: (1/2*pi) integral(kx.cos^2) = 1/2.
theorem parseval_cos_k(k: Nat) {
    mean_integral(pointwise_mul(cos_nt(k.suc), cos_nt(k.suc))) = Real.one_half
} by {
    mean_integral(pointwise_mul(cos_nt(k.suc), cos_nt(k.suc))) =
        integral(pointwise_mul(cos_nt(k.suc), cos_nt(k.suc)), Real.0, two_pi) / two_pi
    integral_cos_nt_sq(k)
    integral(pointwise_mul(cos_nt(k.suc), cos_nt(k.suc)), Real.0, two_pi) = pi
    mean_integral(pointwise_mul(cos_nt(k.suc), cos_nt(k.suc))) = pi / two_pi
    pi_div_two_pi
    pi / two_pi = Real.one_half
    mean_integral(pointwise_mul(cos_nt(k.suc), cos_nt(k.suc))) = Real.one_half
}

// ---------------------------------------------------------------------------
// Not attempted here (documented limitations)
// ---------------------------------------------------------------------------
//
// The following standard Fourier results are left for future work, each
// blocked by a specific gap in the underlying library:
//
// 1. Linearity of the Fourier coefficients a_n(f + g) = a_n(f) + a_n(g) and
//    a_n(c f) = c a_n(f) for general f and g.  The Darboux integral in
//    `real/integral.ac` has no additivity or scalar-linearity theorem (it is
//    listed there as future work), and the FTC route needs an antiderivative
//    of f * nx.cos for every pair, so only the concrete instances (the
//    coefficients of the harmonics themselves, above) are proved here.
//
// 2. The convolution identity S_n(f) = f * D_n, i.e.
//    S_n(f)(x) = (1/2*pi) integral(f(t) D_n(x - t), 0, 2*pi).  It needs the
//    substitution x - t inside the Darboux integral (a change-of-variables
//    rule), which does not exist in the library; the identity was therefore
//    proved only for the trigonometric kernel itself (the mean of D_n is one,
//    `dirichlet_kernel_mean_one`).
//
// 3. The general orthogonality of distinct harmonics:
//    integral(mx.sin nx.sin) = 0 for m != n, and the mixed cases.  The
//    product-to-sum identities reduce these to the integrals of ((m - n)x).cos
//    and ((m + n)x).cos, but the natural subtraction m - n truncates in Nat,
//    and the case split m < n / m = n / m > n is needed; the same
//    substitution rule as in (2) would be the cleaner route.  What is proved
//    here instead: the diagonal normalization integral(kx.sin^2) = pi,
//    integral(kx.cos^2) = pi and the same-frequency cross term
//    integral(kx.sin kx.cos) = 0 for every k >= 1 (the successor
//    formulation `integral_sin_nt_sq`, `integral_cos_nt_sq`,
//    `integral_sin_nt_cos_nt`).
//
// 4. The Riemann-Lebesgue lemma: integral(f(x) nx.cos) -> 0 as n -> infinity
//    for integrable f.  This needs an oscillation/step-approximation argument
//    over the Darboux sums, and a limit statement over n; neither the
//    approximation machinery nor the connection between the Darboux integral
//    and the limit n -> infinity exists yet in the library.
