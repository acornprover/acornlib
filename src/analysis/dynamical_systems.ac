/// Dynamical systems foundations on the real line: iterated maps, fixed
/// points, and convergence.
///
/// This file collects the classical concrete one-dimensional dynamical
/// systems and the fixed-point machinery around them:
///
///   * the logistic map x ↦ r·x·(1 − x) and its fixed points 0 and 1 − 1/r,
///   * the contraction fixed point (the one-dimensional Banach fixed point
///     theorem): a map with Lipschitz constant c < 1 has at most one fixed
///     point, and existence follows from completeness (stated),
///   * the Babylonian iteration x_{n+1} = (x_n + a/x_n)/2 for square roots,
///     whose step fixes √a,
///   * the tent map T(x) = 2x for x ≤ 1/2 and T(x) = 2(1 − x) for x > 1/2,
///     with T(1/4) = T(3/4) = 1/2,
///   * the derivative criterion for stability of fixed points (stated).
///
/// The generic iteration machinery (`iterate`, `is_fixed_point`) lives in
/// `analysis.dynamical.dynamical_systems`; this file instantiates it on the
/// concrete maps.

from nat import Nat
from order import lt_imp_lte, lte_antisymm, lt_imp_ne
from real import Real, two, one_half_positive, one_half_plus_one_half, add_zero_left,
    add_zero_right, add_assoc, add_neg_eq_zero, neg_neg, neg_distrib,
    lt_add_left, lt_add_right, lte_add_right, mul_zero_left, mul_zero_right,
    mul_one_left, mul_one_right, mul_distrib_left, mul_distrib_right, mul_sub_distrib_left,
    mul_sub_distrib_right, mul_inverse, real_mul_comm, mul_assoc, gt_zero_imp_pos,
    pos_gt_zero, mul_pos_pos, lt_mul_pos_right, lt_imp_minus_pos, gt_imp_not_lte,
    abs_gte_zero, only_abs_zero_eq_zero, sub_zero_imp_eq, sub_cancels, div_le_of_mul_le,
    sqrt_mul_self
from analysis.dynamical.dynamical_systems import iterate, iterate_zero, iterate_suc, is_fixed_point

numerals Real

// =============================================================================
// The logistic map
// =============================================================================

/// The logistic map with parameter r: x ↦ r·x·(1 − x).
define logistic_map(r: Real, x: Real) -> Real {
    r * x * (Real.1 - x)
}

/// The logistic map sends zero to zero: f(0) = 0.
theorem logistic_map_zero(r: Real) {
    logistic_map(r, Real.0) = Real.0
} by {
    logistic_map(r, Real.0) = r * Real.0 * (Real.1 - Real.0)
    mul_zero_right(r)
    r * Real.0 = Real.0
    Real.1 - Real.0 = Real.1
    mul_zero_left(Real.1)
    Real.0 * Real.1 = Real.0
    r * Real.0 * (Real.1 - Real.0) = Real.0
    logistic_map(r, Real.0) = Real.0
}

/// Zero is a fixed point of the logistic map.
theorem logistic_map_zero_fixed_point(r: Real) {
    is_fixed_point(logistic_map(r), Real.0)
} by {
    logistic_map_zero(r)
    is_fixed_point(logistic_map(r), Real.0) = (logistic_map(r, Real.0) = Real.0)
    is_fixed_point(logistic_map(r), Real.0)
}

/// Subtracting (1 − a) from 1 leaves a: 1 − (1 − a) = a.
theorem one_sub_sub_cancel(a: Real) {
    Real.1 - (Real.1 - a) = a
} by {
    Real.1 - (Real.1 - a) = Real.1 + -(Real.1 - a)
    Real.1 - a = Real.1 + -a
    -(Real.1 - a) = -(Real.1 + -a)
    neg_distrib(Real.1, -a)
    -(Real.1 + -a) = -Real.1 + -(-a)
    neg_neg(a)
    -(-a) = a
    -(Real.1 - a) = -Real.1 + a
    Real.1 + -(Real.1 - a) = Real.1 + (-Real.1 + a)
    add_assoc(Real.1, -Real.1, a)
    Real.1 + (-Real.1 + a) = (Real.1 + -Real.1) + a
    add_neg_eq_zero(Real.1)
    Real.1 + -Real.1 = Real.0
    (Real.1 + -Real.1) + a = Real.0 + a
    add_zero_left(a)
    Real.0 + a = a
    Real.1 + -(Real.1 - a) = a
    Real.1 - (Real.1 - a) = a
}

/// The nonzero logistic fixed point: f(1 − 1/r) = 1 − 1/r for r ≠ 0.
///
/// The algebra: r·(1 − 1/r) = r − 1, while 1 − (1 − 1/r) = 1/r, so
/// f(1 − 1/r) = (r − 1)·(1/r) = 1 − 1/r.
theorem logistic_map_second_fixed_point(r: Real) {
    r != Real.0 implies logistic_map(r, Real.1 - Real.1 / r) = Real.1 - Real.1 / r
} by {
    if r != Real.0 {
        mul_inverse(r)
        r * r.inverse = Real.1
        Real.1 / r = Real.1 * r.inverse
        mul_one_left(r.inverse)
        Real.1 * r.inverse = r.inverse
        Real.1 / r = r.inverse
        mul_sub_distrib_left(r, Real.1, Real.1 / r)
        r * (Real.1 - Real.1 / r) = r * Real.1 - r * (Real.1 / r)
        mul_one_right(r)
        r * Real.1 = r
        r * (Real.1 / r) = r * r.inverse
        r * (Real.1 / r) = Real.1
        r * (Real.1 - Real.1 / r) = r - Real.1
        one_sub_sub_cancel(Real.1 / r)
        Real.1 - (Real.1 - Real.1 / r) = Real.1 / r
        logistic_map(r, Real.1 - Real.1 / r) =
            r * (Real.1 - Real.1 / r) * (Real.1 - (Real.1 - Real.1 / r))
        logistic_map(r, Real.1 - Real.1 / r) = (r - Real.1) * (Real.1 / r)
        mul_sub_distrib_right(r, Real.1, r.inverse)
        (r - Real.1) * r.inverse = r * r.inverse - Real.1 * r.inverse
        (r - Real.1) * r.inverse = Real.1 - r.inverse
        (r - Real.1) * (Real.1 / r) = (r - Real.1) * r.inverse
        (r - Real.1) * (Real.1 / r) = Real.1 - r.inverse
        Real.1 - r.inverse = Real.1 - Real.1 / r
        (r - Real.1) * (Real.1 / r) = Real.1 - Real.1 / r
        logistic_map(r, Real.1 - Real.1 / r) = Real.1 - Real.1 / r
    }
}

/// The point 1 − 1/r is a fixed point of the logistic map when r ≠ 0.
theorem logistic_map_second_fixed_point_fp(r: Real) {
    r != Real.0 implies is_fixed_point(logistic_map(r), Real.1 - Real.1 / r)
} by {
    if r != Real.0 {
        logistic_map_second_fixed_point(r)
        is_fixed_point(logistic_map(r), Real.1 - Real.1 / r) =
            (logistic_map(r, Real.1 - Real.1 / r) = Real.1 - Real.1 / r)
        is_fixed_point(logistic_map(r), Real.1 - Real.1 / r)
    }
}

/// The logistic fixed points are the only fixed points: any solution of
/// r·x·(1 − x) = x with r ≠ 0 and r ≠ 1 is x = 1 − 1/r (besides x = 0).
///
/// The converse is stated below for future work; the derivation factors x
/// out of r·x·(1 − x) − x and uses r·(1 − x) = 1 to solve for x.
// theorem logistic_map_fixed_points_only(r: Real, x: Real) {
//     r != Real.0 and r != Real.1 and logistic_map(r, x) = x
//     implies x = Real.0 or x = Real.1 - Real.1 / r
// }

// =============================================================================
// The contraction fixed point (one-dimensional Banach fixed point theorem)
// =============================================================================

/// True if f is a contraction of the real line with constant c: the constant
/// is at least zero, strictly below one, and bounds the distortion of
/// distances: |f(x) − f(y)| ≤ c·|x − y| for all x and y.
define is_contraction(f: Real -> Real, c: Real) -> Bool {
    Real.0 <= c and c < Real.1
        and forall(x: Real, y: Real) {
            (f(x) - f(y)).abs <= c * (x - y).abs
        }
}

/// A contraction satisfies its Lipschitz inequality at each pair of points.
theorem is_contraction_apply(f: Real -> Real, c: Real, x: Real, y: Real) {
    is_contraction(f, c) implies (f(x) - f(y)).abs <= c * (x - y).abs
} by {
    if is_contraction(f, c) {
        is_contraction(f, c) =
            (Real.0 <= c and c < Real.1
                and forall(u: Real, v: Real) { (f(u) - f(v)).abs <= c * (u - v).abs })
        forall(u: Real, v: Real) { (f(u) - f(v)).abs <= c * (u - v).abs }
        (f(x) - f(y)).abs <= c * (x - y).abs
    }
}

/// The Lipschitz inequality, the nonnegativity of the constant, and the
/// strict bound c < 1 together make f a contraction.
theorem is_contraction_intro(f: Real -> Real, c: Real) {
    Real.0 <= c and c < Real.1
        and (forall(x: Real, y: Real) { (f(x) - f(y)).abs <= c * (x - y).abs })
    implies is_contraction(f, c)
} by {
    if Real.0 <= c and c < Real.1
        and (forall(x: Real, y: Real) { (f(x) - f(y)).abs <= c * (x - y).abs }) {
        is_contraction(f, c) =
            (Real.0 <= c and c < Real.1
                and forall(u: Real, v: Real) { (f(u) - f(v)).abs <= c * (u - v).abs })
        is_contraction(f, c)
    }
}

/// The contraction constant is strictly below one.
theorem is_contraction_constant_lt_one(f: Real -> Real, c: Real) {
    is_contraction(f, c) implies c < Real.1
} by {
    if is_contraction(f, c) {
        is_contraction(f, c) =
            (Real.0 <= c and c < Real.1
                and forall(u: Real, v: Real) { (f(u) - f(v)).abs <= c * (u - v).abs })
        c < Real.1
    }
}

/// The contraction constant is nonnegative.
theorem is_contraction_constant_nonneg(f: Real -> Real, c: Real) {
    is_contraction(f, c) implies Real.0 <= c
} by {
    if is_contraction(f, c) {
        is_contraction(f, c) =
            (Real.0 <= c and c < Real.1
                and forall(u: Real, v: Real) { (f(u) - f(v)).abs <= c * (u - v).abs })
        Real.0 <= c
    }
}

/// A contraction has at most one fixed point: two fixed points of f coincide.
///
/// The argument: with d = |x − y|, the Lipschitz inequality at the two fixed
/// points gives d ≤ c·d, so d·(1 − c) ≤ 0; since 1 − c > 0 this forces d ≤ 0,
/// and d ≥ 0 always, so d = 0 and x = y.
theorem contraction_fixed_point_unique(f: Real -> Real, c: Real, x: Real, y: Real) {
    is_contraction(f, c) and f(x) = x and f(y) = y implies x = y
} by {
    if is_contraction(f, c) and f(x) = x and f(y) = y {
        is_contraction_apply(f, c, x, y)
        (f(x) - f(y)).abs <= c * (x - y).abs
        f(x) = x
        f(y) = y
        (f(x) - f(y)).abs = (x - y).abs
        (x - y).abs <= c * (x - y).abs
        lte_add_right((x - y).abs, c * (x - y).abs, -(c * (x - y).abs))
        (x - y).abs + -(c * (x - y).abs) <= c * (x - y).abs + -(c * (x - y).abs)
        c * (x - y).abs + -(c * (x - y).abs) = Real.0
        (x - y).abs + -(c * (x - y).abs) = (x - y).abs - c * (x - y).abs
        (x - y).abs - c * (x - y).abs <= Real.0
        mul_sub_distrib_right((x - y).abs, Real.1, c)
        (x - y).abs * (Real.1 - c) = (x - y).abs * Real.1 - (x - y).abs * c
        mul_one_right((x - y).abs)
        (x - y).abs * Real.1 = (x - y).abs
        real_mul_comm((x - y).abs, c)
        (x - y).abs * c = c * (x - y).abs
        (x - y).abs - c * (x - y).abs = (x - y).abs * (Real.1 - c)
        (x - y).abs * (Real.1 - c) <= Real.0
        is_contraction_constant_lt_one(f, c)
        c < Real.1
        lt_imp_minus_pos(c, Real.1)
        (Real.1 - c).is_positive
        pos_gt_zero(Real.1 - c)
        Real.1 - c > Real.0
        div_le_of_mul_le((x - y).abs, Real.1 - c, Real.0)
        (x - y).abs <= Real.0 / (Real.1 - c)
        Real.0 / (Real.1 - c) = Real.0 * (Real.1 - c).inverse
        mul_zero_left((Real.1 - c).inverse)
        Real.0 * (Real.1 - c).inverse = Real.0
        Real.0 / (Real.1 - c) = Real.0
        (x - y).abs <= Real.0
        abs_gte_zero(x - y)
        (x - y).abs >= Real.0
        Real.0 <= (x - y).abs
        lte_antisymm((x - y).abs, Real.0)
        (x - y).abs = Real.0
        only_abs_zero_eq_zero(x - y)
        x - y = Real.0
        sub_zero_imp_eq(x, y)
        x = y
    }
}

// The one-dimensional Banach fixed point theorem: a contraction of the real
// line has a fixed point.  Existence is the constructive completeness
// argument: pick any x0; the iterates f^n(x0) form a Cauchy sequence because
// |f^{n+1}(x0) − f^n(x0)| ≤ c^n·|f(x0) − x0| and the geometric series
// converges, so by `cauchy_imp_exists_limit` the iterates converge to some
// xstar; the Lipschitz inequality makes f continuous, and passing the limit
// through x_{n+1} = f(x_n) gives f(xstar) = xstar.  Combined with
// `contraction_fixed_point_unique` this yields a unique fixed point.  The
// completeness and continuity pieces exist in the library (is_cauchy_seq,
// cauchy_imp_exists_limit, continuous); assembling the full argument is left
// for the metric-space development in `analysis.metric`.
//
// theorem banach_contraction_fixed_point(f: Real -> Real, c: Real) {
//     is_contraction(f, c) implies exists(xstar: Real) {
//         f(xstar) = xstar and forall(z: Real) { f(z) = z implies z = xstar }
//     }
// }

// =============================================================================
// The Babylonian method for square roots
// =============================================================================

/// The Babylonian step for a square root of a: x ↦ (x + a/x)/2.
define babylonian_step(a: Real, x: Real) -> Real {
    (x + a / x) * Real.one_half
}

/// The Babylonian iteration: the sequence x_{n+1} = (x_n + a/x_n)/2 starting
/// at x0.
define babylonian_seq(a: Real, x0: Real, n: Nat) -> Real {
    iterate(babylonian_step(a), n, x0)
}

/// The zeroth Babylonian iterate is the initial guess.
theorem babylonian_seq_zero(a: Real, x0: Real) {
    babylonian_seq(a, x0, Nat.0) = x0
} by {
    iterate_zero(babylonian_step(a), x0)
}

/// Each Babylonian iterate is the step applied to the previous one.
theorem babylonian_seq_suc(a: Real, x0: Real, n: Nat) {
    babylonian_seq(a, x0, n.suc) = babylonian_step(a, babylonian_seq(a, x0, n))
} by {
    iterate_suc(babylonian_step(a), n, x0)
    babylonian_seq(a, x0, n.suc) = babylonian_step(a, iterate(babylonian_step(a), n, x0))
    babylonian_step(a, iterate(babylonian_step(a), n, x0)) =
        babylonian_step(a, babylonian_seq(a, x0, n))
    babylonian_seq(a, x0, n.suc) = babylonian_step(a, babylonian_seq(a, x0, n))
}

/// The Babylonian step fixes every square root of a: if y² = a and y ≠ 0 then
/// (y + a/y)/2 = y.
///
/// The algebra: a/y = y²/y = y, so the step becomes (y + y)/2 = y.  This is
/// the fixed-point identity f(√a) = √a: `sqrt_mul_self` supplies a witness y
/// with y² = a for every a ≥ 0, so `babylonian_has_fixed_point` below turns
/// this identity into the existence of a square-root witness fixed by the
/// step.
theorem babylonian_step_fixed_point(a: Real, y: Real) {
    y * y = a and y != Real.0 implies babylonian_step(a, y) = y
} by {
    if y * y = a and y != Real.0 {
        mul_inverse(y)
        y * y.inverse = Real.1
        mul_assoc(y, y, y.inverse)
        y * (y * y.inverse) = (y * y) * y.inverse
        y * (y * y.inverse) = y * Real.1
        mul_one_right(y)
        y * Real.1 = y
        (y * y) * y.inverse = y
        a / y = a * y.inverse
        a * y.inverse = (y * y) * y.inverse
        a * y.inverse = y
        a / y = y
        babylonian_step(a, y) = (y + a / y) * Real.one_half
        (y + a / y) * Real.one_half = (y + y) * Real.one_half
        two = Real.1 + Real.1
        mul_distrib_right(y, Real.1, Real.1)
        y * (Real.1 + Real.1) = y * Real.1 + y * Real.1
        mul_one_right(y)
        y * Real.1 = y
        y * two = y + y
        mul_assoc(y, two, Real.one_half)
        y * (two * Real.one_half) = (y * two) * Real.one_half
        (y + y) * Real.one_half = y * (two * Real.one_half)
        mul_distrib_left(Real.1, Real.1, Real.one_half)
        (Real.1 + Real.1) * Real.one_half = Real.one_half + Real.one_half
        one_half_plus_one_half
        Real.one_half + Real.one_half = Real.1
        two * Real.one_half = Real.1
        y * (two * Real.one_half) = y * Real.1
        y * (two * Real.one_half) = y
        (y + y) * Real.one_half = y
        babylonian_step(a, y) = y
    }
}

/// Every positive a has a square-root witness fixed by the Babylonian step.
theorem babylonian_has_fixed_point(a: Real) {
    a > Real.0 implies exists(y: Real) {
        y != Real.0 and y * y = a and babylonian_step(a, y) = y
    }
} by {
    if a > Real.0 {
        lt_imp_lte(Real.0, a)
        Real.0 <= a
        sqrt_mul_self(a)
        exists(y: Real) { a.sqrt = Option.some(y) and y * y = a }
        let y: Real satisfy {
            a.sqrt = Option.some(y) and y * y = a
        }
        y * y = a
        if y = Real.0 {
            y * y = Real.0 * Real.0
            mul_zero_left(Real.0)
            Real.0 * Real.0 = Real.0
            y * y = Real.0
            a = Real.0
            lt_imp_ne(Real.0, a)
            Real.0 != a
            false
        }
        y != Real.0
        babylonian_step_fixed_point(a, y)
        babylonian_step(a, y) = y
        exists(w: Real) {
            w != Real.0 and w * w = a and babylonian_step(a, w) = w
        }
    }
}

// The Babylonian iteration converges to the square root of a: for a > 0 and
// x0 > 0 the sequence x_{n+1} = (x_n + a/x_n)/2 converges to the unique
// positive y with y² = a.  The standard proof bounds the error
// x_{n+1} − y by (x_n − y)²/(2·x_n): the step is a contraction-like map near
// the fixed point, so the iterates form a Cauchy sequence and converge to the
// fixed point y.  The full convergence argument is left for the Banach /
// completeness development above.
//
// theorem babylonian_seq_converges(a: Real, x0: Real) {
//     a > Real.0 and x0 > Real.0 implies exists(y: Real) {
//         y * y = a and y >= Real.0 and converges_to(babylonian_seq(a, x0), y)
//     }
// }

// =============================================================================
// The tent map
// =============================================================================

/// One quarter: (1/2)² = 1/4.
let quarter: Real = Real.one_half * Real.one_half

/// Three quarters: 1 − 1/4 = 3/4.
let three_quarters: Real = Real.1 - quarter

/// The tent map: T(x) = 2x for x ≤ 1/2, and T(x) = 2(1 − x) for x > 1/2.
define tent(x: Real) -> Real {
    if x <= Real.one_half {
        two * x
    } else {
        two * (Real.1 - x)
    }
}

/// One half is less than one.
theorem one_half_lt_one {
    Real.one_half < Real.1
} by {
    one_half_positive
    Real.0 < Real.one_half
    one_half_plus_one_half
    Real.one_half + Real.one_half = Real.1
    lt_add_right(Real.0, Real.one_half, Real.one_half)
    Real.0 + Real.one_half < Real.one_half + Real.one_half
    add_zero_left(Real.one_half)
    Real.0 + Real.one_half = Real.one_half
    Real.one_half < Real.1
}

/// One quarter lies below one half.
theorem quarter_lte_one_half {
    quarter <= Real.one_half
} by {
    one_half_lt_one
    Real.one_half < Real.1
    one_half_positive
    Real.0 < Real.one_half
    gt_zero_imp_pos(Real.one_half)
    Real.one_half.is_positive
    lt_mul_pos_right(Real.one_half, Real.1, Real.one_half)
    Real.one_half * Real.one_half < Real.1 * Real.one_half
    mul_one_left(Real.one_half)
    Real.1 * Real.one_half = Real.one_half
    Real.one_half * Real.one_half < Real.one_half
    lt_imp_lte(Real.one_half * Real.one_half, Real.one_half)
    quarter <= Real.one_half
}

/// Twice a quarter is one half: 2·(1/4) = 1/2.
theorem two_mul_quarter_eq_one_half {
    two * quarter = Real.one_half
} by {
    quarter = Real.one_half * Real.one_half
    two = Real.1 + Real.1
    mul_assoc(two, Real.one_half, Real.one_half)
    two * (Real.one_half * Real.one_half) = (two * Real.one_half) * Real.one_half
    mul_distrib_left(Real.1, Real.1, Real.one_half)
    (Real.1 + Real.1) * Real.one_half = Real.one_half + Real.one_half
    one_half_plus_one_half
    Real.one_half + Real.one_half = Real.1
    two * Real.one_half = Real.1
    mul_one_left(Real.one_half)
    Real.1 * Real.one_half = Real.one_half
    two * quarter = Real.one_half
}

/// The tent map at one quarter is one half: T(1/4) = 1/2.
theorem tent_quarter {
    tent(quarter) = Real.one_half
} by {
    quarter_lte_one_half
    if quarter <= Real.one_half {
        two_mul_quarter_eq_one_half
        two * quarter = Real.one_half
        tent(quarter) = two * quarter
        tent(quarter) = Real.one_half
    } else {
        false
    }
}

/// One quarter is positive.
theorem quarter_positive {
    quarter > Real.0
} by {
    one_half_positive
    Real.0 < Real.one_half
    gt_zero_imp_pos(Real.one_half)
    Real.one_half.is_positive
    mul_pos_pos(Real.one_half, Real.one_half)
    Real.one_half * Real.one_half > Real.0
    quarter > Real.0
}

/// One half minus one quarter is one quarter: 1/2 − 1/4 = 1/4.
theorem one_half_minus_quarter {
    Real.one_half - quarter = quarter
} by {
    mul_sub_distrib_right(Real.one_half, Real.1, Real.one_half)
    Real.one_half * (Real.1 - Real.one_half) = Real.one_half * Real.1 - Real.one_half * Real.one_half
    mul_one_right(Real.one_half)
    Real.one_half * Real.1 = Real.one_half
    Real.one_half * (Real.1 - Real.one_half) = Real.one_half - quarter
    sub_cancels(Real.one_half, Real.one_half)
    Real.one_half + Real.one_half - Real.one_half = Real.one_half
    one_half_plus_one_half
    Real.one_half + Real.one_half = Real.1
    Real.1 - Real.one_half = Real.one_half
    Real.one_half * (Real.1 - Real.one_half) = Real.one_half * Real.one_half
    Real.one_half * Real.one_half = quarter
    Real.one_half - quarter = quarter
}

/// Three quarters is one half plus one quarter: 3/4 = 1/2 + 1/4.
theorem three_quarters_eq_half_plus_quarter {
    three_quarters = Real.one_half + quarter
} by {
    one_half_plus_one_half
    Real.one_half + Real.one_half = Real.1
    one_half_minus_quarter
    Real.one_half - quarter = quarter
    three_quarters = Real.1 - quarter
    Real.1 - quarter = Real.one_half + Real.one_half - quarter
    Real.one_half + Real.one_half - quarter = Real.one_half + (Real.one_half - quarter)
    Real.one_half + (Real.one_half - quarter) = Real.one_half + quarter
    Real.1 - quarter = Real.one_half + quarter
    three_quarters = Real.one_half + quarter
}

/// Three quarters exceeds one half: 3/4 > 1/2.
theorem three_quarters_gt_one_half {
    Real.one_half < three_quarters
} by {
    quarter_positive
    quarter > Real.0
    lt_add_left(Real.0, quarter, Real.one_half)
    Real.one_half + Real.0 < Real.one_half + quarter
    add_zero_right(Real.one_half)
    Real.one_half + Real.0 = Real.one_half
    Real.one_half < Real.one_half + quarter
    three_quarters_eq_half_plus_quarter
    Real.one_half + quarter = three_quarters
    Real.one_half < three_quarters
}

/// One minus three quarters is one quarter: 1 − 3/4 = 1/4.
theorem one_minus_three_quarters {
    Real.1 - three_quarters = quarter
} by {
    three_quarters = Real.1 - quarter
    one_sub_sub_cancel(quarter)
    Real.1 - (Real.1 - quarter) = quarter
    Real.1 - three_quarters = quarter
}

/// The tent map at three quarters is one half: T(3/4) = 1/2.
theorem tent_three_quarters {
    tent(three_quarters) = Real.one_half
} by {
    three_quarters_gt_one_half
    Real.one_half < three_quarters
    gt_imp_not_lte(three_quarters, Real.one_half)
    not three_quarters <= Real.one_half
    if three_quarters <= Real.one_half {
        false
    } else {
        one_minus_three_quarters
        Real.1 - three_quarters = quarter
        tent(three_quarters) = two * (Real.1 - three_quarters)
        two * (Real.1 - three_quarters) = two * quarter
        two_mul_quarter_eq_one_half
        two * quarter = Real.one_half
        two * (Real.1 - three_quarters) = Real.one_half
        tent(three_quarters) = Real.one_half
    }
}

// =============================================================================
// Stability of fixed points
// =============================================================================

/// True if df is the derivative of f at xstar, in the difference-quotient
/// sense: (f(xstar + h) − f(xstar))/h approaches df(xstar) as h → 0.
define is_derivative_at(f: Real -> Real, df: Real -> Real, xstar: Real) -> Bool {
    forall(eps: Real) {
        eps.is_positive implies exists(delta: Real) {
            delta.is_positive and forall(h: Real) {
                h != Real.0 and h.abs < delta
                implies ((f(xstar + h) - f(xstar)) / h - df(xstar)).abs < eps
            }
        }
    }
}

/// True if xstar is a stable fixed point of f: every orbit starting
/// sufficiently close to xstar stays arbitrarily close forever.
define is_stable_fixed_point(f: Real -> Real, xstar: Real) -> Bool {
    is_fixed_point(f, xstar)
        and forall(eps: Real) {
            eps.is_positive implies exists(delta: Real) {
                delta.is_positive and forall(x0: Real) {
                    (x0 - xstar).abs < delta
                    implies forall(n: Nat) {
                        (iterate(f, n, x0) - xstar).abs < eps
                    }
                }
            }
        }
}

// The derivative criterion for stability: a fixed point xstar of the
// iteration x_{n+1} = f(x_n) is stable when |f'(xstar)| < 1.  This is the
// linearization criterion for one-dimensional dynamical systems; its proof
// uses the mean value theorem to get |f(x) − f(xstar)| ≤ (|f'(xstar)| + η)·|x
// − xstar| on a neighborhood of xstar, making the iteration a contraction
// there, which keeps every nearby orbit inside the neighborhood.
//
// theorem derivative_criterion_for_stability(f: Real -> Real, df: Real -> Real, xstar: Real) {
//     is_derivative_at(f, df, xstar) and is_fixed_point(f, xstar) and (df(xstar)).abs < Real.1
//     implies is_stable_fixed_point(f, xstar)
// }
