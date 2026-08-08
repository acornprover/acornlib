from nat import Nat
from data.basic.set import Set
from data.basic.set_infinite import maps_into, maps_into_intro, set_infinite_of_injection
from data.basic.functions import is_injective_fn, compose

/// True if every value of the parametrisation satisfies the condition.
///
/// The abstract form of a parametric solution family: `sol` assigns a candidate to each value
/// of the parameter, and every candidate is a solution. Nothing here is specific to
/// polynomials, so the same statement covers a polynomial identity, a recurrence, or any
/// other rule that produces solutions from a parameter.
define parametrises_solutions[P, X](cond: X -> Bool, sol: P -> X) -> Bool {
    forall(t: P) {
        cond(sol(t))
    }
}

/// Each value of such a parametrisation is a solution.
theorem parametrises_solutions_apply[P, X](cond: X -> Bool, sol: P -> X, t: P) {
    parametrises_solutions(cond, sol) implies cond(sol(t))
} by {
    if parametrises_solutions(cond, sol) {
        parametrises_solutions(cond, sol) = forall(s: P) {
            cond(sol(s))
        }
        forall(s: P) {
            cond(sol(s))
        }
        cond(sol(t))
    }
}

/// A pointwise check gives a parametrisation.
theorem parametrises_solutions_intro[P, X](cond: X -> Bool, sol: P -> X) {
    (forall(t: P) { cond(sol(t)) }) implies parametrises_solutions(cond, sol)
} by {
    if forall(t: P) { cond(sol(t)) } {
        parametrises_solutions(cond, sol) = forall(s: P) {
            cond(sol(s))
        }
        parametrises_solutions(cond, sol)
    }
}

/// A parametrisation composed with any reindexing is still a parametrisation.
///
/// Restricting the parameter to a subfamily cannot produce a non-solution.
theorem parametrises_solutions_compose[P, Q, X](
    cond: X -> Bool, sol: P -> X, reindex: Q -> P
) {
    parametrises_solutions(cond, sol)
        implies parametrises_solutions(cond, compose(sol, reindex))
} by {
    if parametrises_solutions(cond, sol) {
        forall(t: Q) {
            parametrises_solutions_apply(cond, sol, reindex(t))
            cond(sol(reindex(t)))
            compose(sol, reindex)(t) = sol(reindex(t))
            cond(compose(sol, reindex)(t))
        }
        parametrises_solutions_intro(cond, compose(sol, reindex))
        parametrises_solutions(cond, compose(sol, reindex))
    }
}

/// The set of values satisfying a condition.
define solution_set[X](cond: X -> Bool) -> Set[X] {
    Set[X].new(cond)
}

/// Membership in the solution set is satisfying the condition.
theorem solution_set_contains_eq[X](cond: X -> Bool, x: X) {
    solution_set(cond).contains(x) = cond(x)
} by {
    Set[X].new(cond).contains(x) = cond(x)
}

/// Every value of a parametrisation lies in the solution set.
theorem parametrised_value_in_solution_set[P, X](
    cond: X -> Bool, sol: P -> X, t: P
) {
    parametrises_solutions(cond, sol) implies solution_set(cond).contains(sol(t))
} by {
    if parametrises_solutions(cond, sol) {
        parametrises_solutions_apply(cond, sol, t)
        cond(sol(t))
        solution_set_contains_eq(cond, sol(t))
        solution_set(cond).contains(sol(t))
    }
}

/// An injective parametrisation gives distinct solutions at distinct parameters.
///
/// This is the half that turns a family into a genuinely large one: without injectivity a
/// parametrisation may name the same solution over and over.
theorem injective_parametrisation_distinct[P, X](
    cond: X -> Bool, sol: P -> X, s: P, t: P
) {
    is_injective_fn(sol) and s != t implies sol(s) != sol(t)
} by {
    if is_injective_fn(sol) and s != t {
        is_injective_fn(sol) = forall(a: P, b: P) { sol(a) = sol(b) implies a = b }
        forall(a: P, b: P) { sol(a) = sol(b) implies a = b }
        (sol(s) = sol(t) implies s = t)
        sol(s) != sol(t)
    }
}

/// An injective parametrisation embeds its parameter type in the solution set.
///
/// The two conditions together are what a parametric-solution claim asserts: every parameter
/// gives a solution, and different parameters give different ones. Whatever notion of size
/// the parameter type carries transfers to the solutions.
define embeds_solutions[P, X](cond: X -> Bool, sol: P -> X) -> Bool {
    parametrises_solutions(cond, sol) and is_injective_fn(sol)
}

/// Such an embedding parametrises solutions.
theorem embeds_solutions_parametrises[P, X](cond: X -> Bool, sol: P -> X) {
    embeds_solutions(cond, sol) implies parametrises_solutions(cond, sol)
} by {
    if embeds_solutions(cond, sol) {
        embeds_solutions(cond, sol) =
            (parametrises_solutions(cond, sol) and is_injective_fn(sol))
        parametrises_solutions(cond, sol)
    }
}

/// Such an embedding is injective.
theorem embeds_solutions_injective[P, X](cond: X -> Bool, sol: P -> X) {
    embeds_solutions(cond, sol) implies is_injective_fn(sol)
} by {
    if embeds_solutions(cond, sol) {
        embeds_solutions(cond, sol) =
            (parametrises_solutions(cond, sol) and is_injective_fn(sol))
        is_injective_fn(sol)
    }
}

/// The two conditions give an embedding.
theorem embeds_solutions_intro[P, X](cond: X -> Bool, sol: P -> X) {
    parametrises_solutions(cond, sol) and is_injective_fn(sol)
        implies embeds_solutions(cond, sol)
} by {
    if parametrises_solutions(cond, sol) and is_injective_fn(sol) {
        embeds_solutions(cond, sol) =
            (parametrises_solutions(cond, sol) and is_injective_fn(sol))
        embeds_solutions(cond, sol)
    }
}

/// A weaker condition inherits any parametrisation of a stronger one.
///
/// So a family exhibited for a specific equation also witnesses every consequence of it.
theorem parametrises_solutions_weaken[P, X](
    cond: X -> Bool, weaker: X -> Bool, sol: P -> X
) {
    parametrises_solutions(cond, sol) and (forall(x: X) { cond(x) implies weaker(x) })
        implies parametrises_solutions(weaker, sol)
} by {
    if parametrises_solutions(cond, sol) and forall(x: X) { cond(x) implies weaker(x) } {
        forall(t: P) {
            parametrises_solutions_apply(cond, sol, t)
            cond(sol(t))
            weaker(sol(t))
        }
        parametrises_solutions_intro(weaker, sol)
        parametrises_solutions(weaker, sol)
    }
}

/// An embedding of solutions survives weakening the condition.
theorem embeds_solutions_weaken[P, X](
    cond: X -> Bool, weaker: X -> Bool, sol: P -> X
) {
    embeds_solutions(cond, sol) and (forall(x: X) { cond(x) implies weaker(x) })
        implies embeds_solutions(weaker, sol)
} by {
    if embeds_solutions(cond, sol) and forall(x: X) { cond(x) implies weaker(x) } {
        embeds_solutions_parametrises(cond, sol)
        parametrises_solutions(cond, sol)
        parametrises_solutions_weaken(cond, weaker, sol)
        parametrises_solutions(weaker, sol)
        embeds_solutions_injective(cond, sol)
        is_injective_fn(sol)
        embeds_solutions_intro(weaker, sol)
        embeds_solutions(weaker, sol)
    }
}

/// Composing an embedding with an injective reindexing is again an embedding.
///
/// The usual way a family is exhibited: a parametrisation over the naturals is obtained by
/// restricting one over a larger parameter type along an injection.
theorem embeds_solutions_compose[P, Q, X](
    cond: X -> Bool, sol: P -> X, reindex: Q -> P
) {
    embeds_solutions(cond, sol) and is_injective_fn(reindex)
        implies embeds_solutions(cond, compose(sol, reindex))
} by {
    if embeds_solutions(cond, sol) and is_injective_fn(reindex) {
        embeds_solutions_parametrises(cond, sol)
        parametrises_solutions(cond, sol)
        parametrises_solutions_compose(cond, sol, reindex)
        parametrises_solutions(cond, compose(sol, reindex))
        embeds_solutions_injective(cond, sol)
        is_injective_fn(sol)
        forall(a: Q, b: Q) {
            if compose(sol, reindex)(a) = compose(sol, reindex)(b) {
                sol(reindex(a)) = sol(reindex(b))
                is_injective_fn(sol) = forall(x: P, y: P) { sol(x) = sol(y) implies x = y }
                forall(x: P, y: P) { sol(x) = sol(y) implies x = y }
                (sol(reindex(a)) = sol(reindex(b)) implies reindex(a) = reindex(b))
                reindex(a) = reindex(b)
                is_injective_fn(reindex) = forall(x: Q, y: Q) {
                    reindex(x) = reindex(y) implies x = y
                }
                forall(x: Q, y: Q) { reindex(x) = reindex(y) implies x = y }
                (reindex(a) = reindex(b) implies a = b)
                a = b
            }
            (compose(sol, reindex)(a) = compose(sol, reindex)(b) implies a = b)
        }
        is_injective_fn(compose(sol, reindex)) = forall(x: Q, y: Q) {
            compose(sol, reindex)(x) = compose(sol, reindex)(y) implies x = y
        }
        is_injective_fn(compose(sol, reindex))
        embeds_solutions_intro(cond, compose(sol, reindex))
        embeds_solutions(cond, compose(sol, reindex))
    }
}

/// A family of solutions indexed injectively by the naturals makes the solution set infinite.
///
/// The capstone of the pattern: exhibiting such a family is what a parametric-solution claim
/// is for. It rests on `set_infinite_of_injection`, which is the criterion `src/set.ac` was
/// missing.
theorem embeds_solutions_infinite[X](cond: X -> Bool, sol: Nat -> X) {
    embeds_solutions(cond, sol) implies solution_set(cond).is_infinite
} by {
    if embeds_solutions(cond, sol) {
        embeds_solutions_parametrises(cond, sol)
        parametrises_solutions(cond, sol)
        forall(i: Nat) {
            parametrised_value_in_solution_set(cond, sol, i)
            solution_set(cond).contains(sol(i))
        }
        maps_into_intro(sol, solution_set(cond))
        maps_into(sol, solution_set(cond))
        embeds_solutions_injective(cond, sol)
        is_injective_fn(sol)
        set_infinite_of_injection(sol, solution_set(cond))
        solution_set(cond).is_infinite
    }
}

/// A condition with a solution family indexed by the naturals has infinitely many solutions.
///
/// The form a downstream claim states: all that must be exhibited is a map from the naturals
/// producing solutions, together with its injectivity.
theorem infinitely_many_solutions[X](cond: X -> Bool, sol: Nat -> X) {
    (forall(t: Nat) { cond(sol(t)) }) and is_injective_fn(sol)
        implies solution_set(cond).is_infinite
} by {
    if (forall(t: Nat) { cond(sol(t)) }) and is_injective_fn(sol) {
        parametrises_solutions_intro(cond, sol)
        parametrises_solutions(cond, sol)
        embeds_solutions_intro(cond, sol)
        embeds_solutions(cond, sol)
        embeds_solutions_infinite(cond, sol)
        solution_set(cond).is_infinite
    }
}
