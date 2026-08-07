from int import Int, div_trans, div_from_nat, gcd_div_left, gcd_div_right
from list import List
from nat import Nat, divides_trans, divides_self
from pair import Pair
from zmod import int_mod_rel
from number_theory import congruence_class_contains, lcm_divides_left, lcm_divides_right
from data.int.int_divisibility import int_divides_sub, int_sub_sub_cancel
from residue_class_disjoint import classes_disjoint, classes_disjoint_intro,
    class_modulus_divides_difference
from residue_class_moduli import all_moduli_divide, all_moduli_divide_head,
    all_moduli_divide_tail, all_moduli_divide_cons_intro, all_moduli_divide_nil

numerals Nat

/// Two congruence classes that meet have a common member equidistant from both residues.
///
/// Each modulus divides the gap between the shared member and its own residue, so their
/// greatest common divisor divides both gaps, and hence the gap between the residues. This is
/// the elementary half of the disjointness criterion; the converse needs the two-modulus
/// Chinese remainder theorem to produce a common member.
theorem meeting_classes_gcd_divides_residue_gap(
    cl1: Pair[Nat, Nat], cl2: Pair[Nat, Nat], x: Int
) {
    congruence_class_contains(cl1, x) and congruence_class_contains(cl2, x)
        implies Int.from_nat(cl1.first).gcd(Int.from_nat(cl2.first)).divides(
            Int.from_nat(cl1.second) - Int.from_nat(cl2.second))
} by {
    if congruence_class_contains(cl1, x) and congruence_class_contains(cl2, x) {
        class_modulus_divides_difference(cl1, x)
        Int.from_nat(cl1.first).divides(x - Int.from_nat(cl1.second))
        class_modulus_divides_difference(cl2, x)
        Int.from_nat(cl2.first).divides(x - Int.from_nat(cl2.second))
        gcd_div_left(Int.from_nat(cl1.first), Int.from_nat(cl2.first))
        (Int.from_nat(cl1.first).gcd(Int.from_nat(cl2.first))
            .divides(Int.from_nat(cl1.first)))
        div_trans(Int.from_nat(cl1.first).gcd(Int.from_nat(cl2.first)),
            Int.from_nat(cl1.first), x - Int.from_nat(cl1.second))
        (Int.from_nat(cl1.first).gcd(Int.from_nat(cl2.first))
            .divides(x - Int.from_nat(cl1.second)))
        gcd_div_right(Int.from_nat(cl1.first), Int.from_nat(cl2.first))
        (Int.from_nat(cl1.first).gcd(Int.from_nat(cl2.first))
            .divides(Int.from_nat(cl2.first)))
        div_trans(Int.from_nat(cl1.first).gcd(Int.from_nat(cl2.first)),
            Int.from_nat(cl2.first), x - Int.from_nat(cl2.second))
        (Int.from_nat(cl1.first).gcd(Int.from_nat(cl2.first))
            .divides(x - Int.from_nat(cl2.second)))
        int_divides_sub(Int.from_nat(cl1.first).gcd(Int.from_nat(cl2.first)),
            x - Int.from_nat(cl2.second), x - Int.from_nat(cl1.second))
        (Int.from_nat(cl1.first).gcd(Int.from_nat(cl2.first))
            .divides((x - Int.from_nat(cl2.second)) - (x - Int.from_nat(cl1.second))))
        int_sub_sub_cancel(x, Int.from_nat(cl1.second), Int.from_nat(cl2.second))
        ((x - Int.from_nat(cl2.second)) - (x - Int.from_nat(cl1.second))
            = Int.from_nat(cl1.second) - Int.from_nat(cl2.second))
        (Int.from_nat(cl1.first).gcd(Int.from_nat(cl2.first))
            .divides(Int.from_nat(cl1.second) - Int.from_nat(cl2.second)))
    }
}

/// Classes whose moduli have a greatest common divisor missing the residue gap are disjoint.
///
/// The usable form of the criterion: a failure of divisibility rules out any common member.
theorem classes_disjoint_of_gcd_not_divides(cl1: Pair[Nat, Nat], cl2: Pair[Nat, Nat]) {
    not Int.from_nat(cl1.first).gcd(Int.from_nat(cl2.first)).divides(
        Int.from_nat(cl1.second) - Int.from_nat(cl2.second))
        implies classes_disjoint(cl1, cl2)
} by {
    if not Int.from_nat(cl1.first).gcd(Int.from_nat(cl2.first)).divides(
        Int.from_nat(cl1.second) - Int.from_nat(cl2.second)) {
        forall(x: Int) {
            if congruence_class_contains(cl1, x) and congruence_class_contains(cl2, x) {
                meeting_classes_gcd_divides_residue_gap(cl1, cl2, x)
                (Int.from_nat(cl1.first).gcd(Int.from_nat(cl2.first))
                    .divides(Int.from_nat(cl1.second) - Int.from_nat(cl2.second)))
                false
            }
            not (congruence_class_contains(cl1, x) and congruence_class_contains(cl2, x))
        }
        classes_disjoint_intro(cl1, cl2)
        classes_disjoint(cl1, cl2)
    }
}

/// The least common multiple of the moduli of a system.
///
/// `system_modulus` in `crt_list.ac` is the *product* of the moduli, which is a valid common
/// multiple but usually far from the least. This is the period a system actually repeats with.
define system_lcm(system: List[Pair[Nat, Nat]]) -> Nat {
    match system {
        List.nil {
            Nat.1
        }
        List.cons(head, tail) {
            head.first.lcm(system_lcm(tail))
        }
    }
}

/// The empty system has period one.
theorem system_lcm_nil {
    system_lcm(List.nil[Pair[Nat, Nat]]) = Nat.1
}

/// The period of a cons system combines the head modulus with the rest.
theorem system_lcm_cons(head: Pair[Nat, Nat], tail: List[Pair[Nat, Nat]]) {
    system_lcm(List.cons(head, tail)) = head.first.lcm(system_lcm(tail))
}

/// A common multiple of the moduli remains one after enlarging it.
theorem all_moduli_divide_of_divides(
    system: List[Pair[Nat, Nat]], k: Nat, m: Nat
) {
    all_moduli_divide(system, k) and k.divides(m) implies all_moduli_divide(system, m)
} by {
    define p(s: List[Pair[Nat, Nat]]) -> Bool {
        all_moduli_divide(s, k) and k.divides(m) implies all_moduli_divide(s, m)
    }
    all_moduli_divide_nil(m)
    p(List.nil[Pair[Nat, Nat]])
    forall(head: Pair[Nat, Nat], tail: List[Pair[Nat, Nat]]) {
        if p(tail) {
            if all_moduli_divide(List.cons(head, tail), k) and k.divides(m) {
                all_moduli_divide_head(head, tail, k)
                head.first.divides(k)
                divides_trans(head.first, k, m)
                head.first.divides(m)
                all_moduli_divide_tail(head, tail, k)
                all_moduli_divide(tail, k)
                all_moduli_divide(tail, m)
                all_moduli_divide_cons_intro(head, tail, m)
                all_moduli_divide(List.cons(head, tail), m)
            }
            p(List.cons(head, tail))
        }
        (p(tail) implies p(List.cons(head, tail)))
    }
    p(List.nil[Pair[Nat, Nat]]) and forall(head: Pair[Nat, Nat], tail: List[Pair[Nat, Nat]]) {
        p(tail) implies p(List.cons(head, tail))
    }
    List.induction(p)
    p(system)
}

/// Every modulus of a system divides its least common multiple.
///
/// So `system_periodic` applies with this as the period, which is the point of computing it.
theorem all_moduli_divide_system_lcm(system: List[Pair[Nat, Nat]]) {
    all_moduli_divide(system, system_lcm(system))
} by {
    define p(s: List[Pair[Nat, Nat]]) -> Bool {
        all_moduli_divide(s, system_lcm(s))
    }
    all_moduli_divide_nil(system_lcm(List.nil[Pair[Nat, Nat]]))
    p(List.nil[Pair[Nat, Nat]])
    forall(head: Pair[Nat, Nat], tail: List[Pair[Nat, Nat]]) {
        if p(tail) {
            all_moduli_divide(tail, system_lcm(tail))
            lcm_divides_right(head.first, system_lcm(tail))
            system_lcm(tail).divides(head.first.lcm(system_lcm(tail)))
            all_moduli_divide_of_divides(tail, system_lcm(tail),
                head.first.lcm(system_lcm(tail)))
            all_moduli_divide(tail, head.first.lcm(system_lcm(tail)))
            lcm_divides_left(head.first, system_lcm(tail))
            head.first.divides(head.first.lcm(system_lcm(tail)))
            all_moduli_divide_cons_intro(head, tail, head.first.lcm(system_lcm(tail)))
            all_moduli_divide(List.cons(head, tail), head.first.lcm(system_lcm(tail)))
            system_lcm_cons(head, tail)
            all_moduli_divide(List.cons(head, tail), system_lcm(List.cons(head, tail)))
            p(List.cons(head, tail))
        }
        (p(tail) implies p(List.cons(head, tail)))
    }
    p(List.nil[Pair[Nat, Nat]]) and forall(head: Pair[Nat, Nat], tail: List[Pair[Nat, Nat]]) {
        p(tail) implies p(List.cons(head, tail))
    }
    List.induction(p)
    p(system)
}
