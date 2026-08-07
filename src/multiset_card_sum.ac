from nat import Nat
from list import List, add_length, is_permutation, permutation_preserves_length
from data.basic.set import Set
from multiset import Multiset, contains_iff_positive_multiplicity, add_contains_iff,
    filter_contains_iff, from_list_nil
from multiset_list import from_list_cons, from_list_singleton, from_list_append,
    from_list_filter, from_list_contains, from_list_eq_imp_permutation

numerals Nat

/// List-backed cardinality of the multiset represented by a list.
define multiset_card_from_list[T](items: List[T]) -> Nat {
    items.length
}

/// The list-backed cardinality is the length of the representing list.
theorem multiset_card_from_list_eq_length[T](items: List[T]) {
    multiset_card_from_list(items) = items.length
}

/// Predicate saying that a multiset has a list representation of cardinality `n`.
define multiset_has_list_card[T](m: Multiset[T], n: Nat) -> Bool {
    exists(items: List[T]) {
        m = Multiset.from_list(items) and items.length = n
    }
}

/// Any list representation gives a list-backed cardinality witness.
theorem from_list_has_list_card[T](items: List[T]) {
    multiset_has_list_card(Multiset.from_list(items), multiset_card_from_list(items))
} by {
}

/// Equal list-backed multisets have the same list-backed cardinality.
theorem multiset_card_from_list_eq_of_from_list_eq[T](a: List[T], b: List[T]) {
    Multiset.from_list(a) = Multiset.from_list(b) implies
        multiset_card_from_list(a) = multiset_card_from_list(b)
} by {
    if Multiset.from_list(a) = Multiset.from_list(b) {
        from_list_eq_imp_permutation(a, b)
        permutation_preserves_length(a, b)
        multiset_card_from_list(a) = multiset_card_from_list(b)
    }
}

/// Permuting a representing list does not change list-backed cardinality.
theorem multiset_card_from_list_permutation[T](a: List[T], b: List[T]) {
    is_permutation(a, b) implies multiset_card_from_list(a) = multiset_card_from_list(b)
} by {
    if is_permutation(a, b) {
        permutation_preserves_length(a, b)
        multiset_card_from_list(a) = multiset_card_from_list(b)
    }
}

/// The empty multiset has list-backed cardinality zero.
theorem multiset_empty_has_list_card_zero[T] {
    multiset_has_list_card(Multiset.empty[T], Nat.0)
} by {
    from_list_nil[T]
}

/// A singleton multiset has list-backed cardinality one.
theorem multiset_singleton_has_list_card_one[T](item: T) {
    multiset_has_list_card(Multiset.singleton(item), Nat.1)
} by {
    from_list_singleton(item)
    List.nil[T].length = Nat.0
    List.singleton(item).length = Nat.1
}

/// Consing a representative list inserts one copy and increases list-backed cardinality by one.
theorem multiset_card_from_list_cons[T](head: T, tail: List[T]) {
    multiset_card_from_list(List.cons(head, tail)) = multiset_card_from_list(tail).suc
} by {
    List.cons(head, tail).length = tail.length.suc
}

/// Inserting an element into a list-backed multiset increases its cardinality witness by one.
theorem multiset_insert_has_list_card_suc[T](m: Multiset[T], item: T, n: Nat) {
    multiset_has_list_card(m, n) implies multiset_has_list_card(m.insert(item), n.suc)
} by {
    if multiset_has_list_card(m, n) {
        let items: List[T] satisfy {
            m = Multiset.from_list(items) and items.length = n
        }
        from_list_cons(item, items)
        List.cons(item, items).length = n.suc
        exists(xs: List[T]) {
            m.insert(item) = Multiset.from_list(xs) and xs.length = n.suc
        }
    }
}

/// Appending representative lists adds list-backed cardinalities.
theorem multiset_card_from_list_append[T](left: List[T], right: List[T]) {
    multiset_card_from_list(left + right) = multiset_card_from_list(left) + multiset_card_from_list(right)
} by {
    add_length(left, right)
}

/// Filtering a representative list gives a cardinality witness for the filtered multiset.
theorem multiset_filter_has_list_card_from_filter[T](items: List[T], pred: T -> Bool) {
    multiset_has_list_card(Multiset.from_list(items).filter(pred), items.filter(pred).length)
} by {
    from_list_filter(items, pred)
}

/// Filtering a list-backed multiset preserves list-backed representability.
theorem multiset_filter_preserves_list_card[T](m: Multiset[T], pred: T -> Bool, n: Nat) {
    multiset_has_list_card(m, n) implies exists(k: Nat) {
        multiset_has_list_card(m.filter(pred), k)
    }
} by {
    if multiset_has_list_card(m, n) {
        let items: List[T] satisfy {
            m = Multiset.from_list(items) and items.length = n
        }
        multiset_filter_has_list_card_from_filter(items, pred)
        exists(k: Nat) {
            k = items.filter(pred).length and multiset_has_list_card(m.filter(pred), k)
        }
    }
}

/// Adding list-backed multisets adds their cardinality witnesses.
theorem multiset_add_has_list_card_add[T](a: Multiset[T], b: Multiset[T], n: Nat, k: Nat) {
    multiset_has_list_card(a, n) and multiset_has_list_card(b, k)
        implies multiset_has_list_card(a + b, n + k)
} by {
    if multiset_has_list_card(a, n) and multiset_has_list_card(b, k) {
        let left: List[T] satisfy {
            a = Multiset.from_list(left) and left.length = n
        }
        let right: List[T] satisfy {
            b = Multiset.from_list(right) and right.length = k
        }
        from_list_append(left, right)
        add_length(left, right)
        (left + right).length = n + k
        exists(items: List[T]) {
            a + b = Multiset.from_list(items) and items.length = n + k
        }
    }
}

/// Membership predicate for the support of a multiset.
define multiset_support_contains[T](m: Multiset[T], item: T) -> Bool {
    m.contains(item)
}

/// The support of a multiset as the set of elements with positive multiplicity.
define multiset_support[T](m: Multiset[T]) -> Set[T] {
    Set[T].new(multiset_support_contains(m))
}

/// Membership in multiset support is ordinary multiset containment.
theorem multiset_support_contains_eq[T](m: Multiset[T], item: T) {
    multiset_support(m).contains(item) = m.contains(item)
} by {
}

/// Support membership is positive multiplicity.
theorem multiset_support_contains_iff_positive_multiplicity[T](m: Multiset[T], item: T) {
    multiset_support(m).contains(item) = (m.multiplicity(item) > Nat.0)
} by {
    multiset_support_contains_eq(m, item)
    contains_iff_positive_multiplicity(m, item)
}

/// The support of a filtered multiset is support intersection with the predicate.
theorem multiset_support_filter_contains[T](m: Multiset[T], pred: T -> Bool, item: T) {
    multiset_support(m.filter(pred)).contains(item) =
        (multiset_support(m).contains(item) and pred(item))
} by {
    multiset_support_contains_eq(m.filter(pred), item)
    multiset_support_contains_eq(m, item)
    filter_contains_iff(m, pred, item)
}

/// The support of a filtered list multiset is list membership together with the predicate.
theorem multiset_support_from_list_filter_contains[T](items: List[T], pred: T -> Bool, item: T) {
    multiset_support(Multiset.from_list(items).filter(pred)).contains(item) =
        (items.contains(item) and pred(item))
} by {
    multiset_support_filter_contains(Multiset.from_list(items), pred, item)
    from_list_contains(items, item)
    multiset_support_contains_eq(Multiset.from_list(items), item)
}

/// The support of a `from_list` multiset is exactly list membership.
theorem multiset_support_from_list_contains[T](items: List[T], item: T) {
    multiset_support(Multiset.from_list(items)).contains(item) = items.contains(item)
} by {
    multiset_support_contains_eq(Multiset.from_list(items), item)
    from_list_contains(items, item)
}

/// Support of multiset addition is the union of supports, pointwise.
theorem multiset_support_add_contains[T](a: Multiset[T], b: Multiset[T], item: T) {
    multiset_support(a + b).contains(item) =
        (multiset_support(a).contains(item) or multiset_support(b).contains(item))
} by {
    multiset_support_contains_eq(a + b, item)
    multiset_support_contains_eq(a, item)
    multiset_support_contains_eq(b, item)
    add_contains_iff(a, b, item)
}
