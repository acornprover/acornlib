from real import Real, lte_trans, abs_gte_zero, mul_pos_pos, square_nonneg, mul_le_mul_nonneg,
    mul_abs, zero_lte_imp_non_neg, abs_of_nonneg, gt_zero_imp_pos, pos_gt_zero
from order import lt_imp_lte
from finite_set import FiniteSet
from data.finite.finite_set_filter import finite_set_filter, finite_set_filter_contains_eq,
    finite_set_filter_mono_pred
from probability.discrete_pmf import DiscretePMF, discrete_expectation, discrete_variance,
    square_dev, square_dev_nonneg, pmf_event_prob, pmf_event_prob_mono
from probability.discrete_probability_ext import discrete_expectation_eq_of_pointwise_eq_on_support
from probability.expectation_properties import rv_ge_contains, rv_ge_event, markov_inequality

/// The square of the absolute value is the square: `|z|^2 = z^2`.
lemma abs_square_eq(z: Real) {
    z.abs * z.abs = z * z
} by {
    mul_abs(z, z)
    z.abs * z.abs = (z * z).abs
    square_nonneg(z)
    z * z >= Real.0
    abs_of_nonneg(z * z)
    (z * z).abs = z * z
    z.abs * z.abs = z * z
}

/// If `t` is nonnegative and `t <= |z|`, then `t^2 <= z^2`.
lemma abs_dev_sq_ge(t: Real, z: Real) {
    Real.0 <= t and t <= z.abs implies t * t <= z * z
} by {
    if Real.0 <= t and t <= z.abs {
        abs_gte_zero(z)
        Real.0 <= z.abs
        mul_le_mul_nonneg(t, t, z.abs, z.abs)
        t * t <= z.abs * z.abs
        abs_square_eq(z)
        z.abs * z.abs = z * z
        t * t <= z * z
    }
}

/// True when the absolute deviation of `rv` from `mu` is at least `t`.
define rv_abs_dev_event_contains[T](rv: T -> Real, mu: Real, t: Real, x: T) -> Bool {
    t <= (rv(x) - mu).abs
}

/// The event that `|X - mu| >= t`, restricted to the pmf support.
define rv_abs_dev_event[T](pmf: DiscretePMF[T], rv: T -> Real, mu: Real, t: Real) -> FiniteSet[T] {
    finite_set_filter(pmf.support, rv_abs_dev_event_contains(rv, mu, t))
}

/// Membership in the `|X - mu| >= t` event is support membership together with
/// the deviation bound.
theorem rv_abs_dev_event_contains_eq[T](pmf: DiscretePMF[T], rv: T -> Real, mu: Real, t: Real, x: T) {
    rv_abs_dev_event(pmf, rv, mu, t).contains(x) =
        (pmf.support.contains(x) and t <= (rv(x) - mu).abs)
} by {
    rv_abs_dev_event_contains(rv, mu, t, x) = (t <= (rv(x) - mu).abs)
    finite_set_filter_contains_eq(pmf.support, rv_abs_dev_event_contains(rv, mu, t), x)
    finite_set_filter(pmf.support, rv_abs_dev_event_contains(rv, mu, t)).contains(x) =
        (pmf.support.contains(x) and rv_abs_dev_event_contains(rv, mu, t, x))
    finite_set_filter(pmf.support, rv_abs_dev_event_contains(rv, mu, t)).contains(x) =
        (pmf.support.contains(x) and t <= (rv(x) - mu).abs)
    rv_abs_dev_event(pmf, rv, mu, t) = finite_set_filter(pmf.support, rv_abs_dev_event_contains(rv, mu, t))
    rv_abs_dev_event(pmf, rv, mu, t).contains(x) = (pmf.support.contains(x) and t <= (rv(x) - mu).abs)
}

/// A point at which `|X - mu| >= t` satisfies `t^2 <= (X - mu)^2` when `t` is
/// nonnegative.
lemma rv_abs_dev_pointwise_sq_ge[T](rv: T -> Real, mu: Real, t: Real, x: T) {
    Real.0 <= t and t <= (rv(x) - mu).abs implies
        t * t <= square_dev(rv, mu, x)
} by {
    if Real.0 <= t and t <= (rv(x) - mu).abs {
        abs_dev_sq_ge(t, rv(x) - mu)
        t * t <= (rv(x) - mu) * (rv(x) - mu)
        square_dev(rv, mu, x) = (rv(x) - mu) * (rv(x) - mu)
        t * t <= square_dev(rv, mu, x)
    }
}

/// The absolute-deviation event `|X - mu| >= t` is contained in the squared-
/// deviation event `(X - mu)^2 >= t^2` when `t` is nonnegative.
theorem rv_abs_dev_event_subset_sq_ge_event[T](pmf: DiscretePMF[T], rv: T -> Real, mu: Real, t: Real) {
    Real.0 <= t implies
        rv_abs_dev_event(pmf, rv, mu, t).subset_eq(rv_ge_event(pmf, square_dev(rv, mu), t * t))
} by {
    if Real.0 <= t {
        forall(x: T) {
            if rv_abs_dev_event_contains(rv, mu, t, x) {
                rv_abs_dev_event_contains(rv, mu, t, x) = (t <= (rv(x) - mu).abs)
                t <= (rv(x) - mu).abs
                rv_abs_dev_pointwise_sq_ge(rv, mu, t, x)
                t * t <= square_dev(rv, mu, x)
                rv_ge_contains(square_dev(rv, mu), t * t, x) = (t * t <= square_dev(rv, mu, x))
                rv_ge_contains(square_dev(rv, mu), t * t, x)
            }
        }
        forall(x: T) {
            rv_abs_dev_event_contains(rv, mu, t, x) implies rv_ge_contains(square_dev(rv, mu), t * t, x)
        }
        finite_set_filter_mono_pred(pmf.support, rv_abs_dev_event_contains(rv, mu, t),
            rv_ge_contains(square_dev(rv, mu), t * t))
        finite_set_filter(pmf.support, rv_abs_dev_event_contains(rv, mu, t)).subset_eq(
            finite_set_filter(pmf.support, rv_ge_contains(square_dev(rv, mu), t * t)))
        rv_abs_dev_event(pmf, rv, mu, t) = finite_set_filter(pmf.support, rv_abs_dev_event_contains(rv, mu, t))
        rv_ge_event(pmf, square_dev(rv, mu), t * t) =
            finite_set_filter(pmf.support, rv_ge_contains(square_dev(rv, mu), t * t))
        rv_abs_dev_event(pmf, rv, mu, t).subset_eq(rv_ge_event(pmf, square_dev(rv, mu), t * t))
    }
}

/// Chebyshev's inequality: if `X` has mean `mu` and `t > 0`, then the
/// probability that `|X - mu| >= t` is at most `Var(X) / t^2`.
///
/// The proof goes through Markov's inequality applied to the nonnegative random
/// variable `(X - mu)^2`, whose expectation is the variance; the event
/// `|X - mu| >= t` is contained in `(X - mu)^2 >= t^2`.
theorem chebyshev_inequality[T](pmf: DiscretePMF[T], rv: T -> Real, mu: Real, t: Real) {
    discrete_expectation(pmf, rv) = mu and t > Real.0 implies
        pmf_event_prob(pmf, rv_abs_dev_event(pmf, rv, mu, t))
            <= discrete_variance(pmf, rv) / (t * t)
} by {
    if discrete_expectation(pmf, rv) = mu and t > Real.0 {
        lt_imp_lte[Real](Real.0, t)
        Real.0 <= t
        rv_abs_dev_event_subset_sq_ge_event(pmf, rv, mu, t)
        rv_abs_dev_event(pmf, rv, mu, t).subset_eq(rv_ge_event(pmf, square_dev(rv, mu), t * t))
        pmf_event_prob_mono(pmf, rv_abs_dev_event(pmf, rv, mu, t),
            rv_ge_event(pmf, square_dev(rv, mu), t * t))
        pmf_event_prob(pmf, rv_abs_dev_event(pmf, rv, mu, t)) <= pmf_event_prob(pmf, rv_ge_event(pmf, square_dev(rv, mu), t * t))

        forall(x: T) {
            if pmf.support.contains(x) {
                square_dev_nonneg(rv, mu, x)
                Real.0 <= square_dev(rv, mu, x)
            }
        }
        forall(x: T) { pmf.support.contains(x) implies Real.0 <= square_dev(rv, mu, x) }
        gt_zero_imp_pos(t)
        t.is_positive
        mul_pos_pos(t, t)
        (t * t).is_positive
        pos_gt_zero(t * t)
        t * t > Real.0
        markov_inequality(pmf, square_dev(rv, mu), t * t)
        pmf_event_prob(pmf, rv_ge_event(pmf, square_dev(rv, mu), t * t)) <= discrete_expectation(pmf, square_dev(rv, mu)) / (t * t)

        forall(x: T) {
            if pmf.support.contains(x) {
                square_dev(rv, mu, x) = (rv(x) - mu) * (rv(x) - mu)
                square_dev(rv, discrete_expectation(pmf, rv), x) =
                    (rv(x) - discrete_expectation(pmf, rv)) * (rv(x) - discrete_expectation(pmf, rv))
                mu = discrete_expectation(pmf, rv)
                square_dev(rv, mu, x) = square_dev(rv, discrete_expectation(pmf, rv), x)
            }
        }
        forall(x: T) {
            pmf.support.contains(x) implies square_dev(rv, mu, x) = square_dev(rv, discrete_expectation(pmf, rv), x)
        }
        discrete_expectation_eq_of_pointwise_eq_on_support(pmf, square_dev(rv, mu),
            square_dev(rv, discrete_expectation(pmf, rv)))
        discrete_expectation(pmf, square_dev(rv, mu)) =
            discrete_expectation(pmf, square_dev(rv, discrete_expectation(pmf, rv)))
        discrete_variance(pmf, rv) =
            discrete_expectation(pmf, square_dev(rv, discrete_expectation(pmf, rv)))
        discrete_expectation(pmf, square_dev(rv, mu)) = discrete_variance(pmf, rv)
        pmf_event_prob(pmf, rv_ge_event(pmf, square_dev(rv, mu), t * t)) <= discrete_variance(pmf, rv) / (t * t)

        lte_trans(pmf_event_prob(pmf, rv_abs_dev_event(pmf, rv, mu, t)),
            pmf_event_prob(pmf, rv_ge_event(pmf, square_dev(rv, mu), t * t)),
            discrete_variance(pmf, rv) / (t * t))
        pmf_event_prob(pmf, rv_abs_dev_event(pmf, rv, mu, t)) <= discrete_variance(pmf, rv) / (t * t)
    }
}

/// Chebyshev's inequality about the mean: `P(|X - E(X)| >= t) <= Var(X) / t^2`
/// for `t > 0`.  This is the form of Chebyshev's inequality most often quoted.
theorem chebyshev_inequality_standard[T](pmf: DiscretePMF[T], rv: T -> Real, t: Real) {
    t > Real.0 implies
        pmf_event_prob(pmf, rv_abs_dev_event(pmf, rv, discrete_expectation(pmf, rv), t))
            <= discrete_variance(pmf, rv) / (t * t)
} by {
    if t > Real.0 {
        chebyshev_inequality(pmf, rv, discrete_expectation(pmf, rv), t)
        discrete_expectation(pmf, rv) = discrete_expectation(pmf, rv)
        pmf_event_prob(pmf, rv_abs_dev_event(pmf, rv, discrete_expectation(pmf, rv), t)) <= discrete_variance(pmf, rv) / (t * t)
    }
}
