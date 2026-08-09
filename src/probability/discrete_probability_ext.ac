from real import Real
from list import List, map, sum, sum_scalar_mul, map_map, scalar_mul, sum_map_of_pointwise,
    list_contains_implies_count_geq_one, unique_implies_tail_unique, unique_implies_no_duplicate
from nat import Nat, sum_lte, lt_suc, lt_and_lte, lte_imp_not_lt
from algebra.semigroup import mul_fn
from algebra.add_semigroup import add_fn
from finite_set import FiniteSet, fs_empty, fs_insert, fs_union, fs_intersection,
    fs_difference, finite_set_ext, finite_set_eq_of_underlying_set_eq,
    finite_set_intersection_assoc, finite_set_intersection_comm,
    finite_set_intersection_contains_eq, finite_set_intersection_idemp,
    finite_set_difference_contains_eq, finite_set_intersection_union_difference_is_self,
    finite_set_intersection_is_disjoint_difference, finite_set_subset_trans,
    finite_set_subset_refl, finite_set_subset_intersection_eq,
    finite_set_sum, finite_set_sum_empty, finite_set_sum_insert,
    finite_set_sum_disjoint_union, finite_set_sum_add, finite_set_sum_scalar_mul
from data.basic.set import Set, set_ext, empty_set_contains_eq, insert_contains_eq,
    intersection_comm, intersection_with_superset_is_self
from data.basic.functions import function_extensionality, compose
from data.finite.finite_set_filter import finite_set_filter, finite_set_filter_contains_eq,
    finite_set_filter_subset
from data.finite.finite_set_sum_product_extra import finite_set_sum_eq_of_pointwise_on_set
from data.finite.finite_fiber_partition import finite_fiber_partition_values,
    finite_fiber_partition_values_spec, finite_fiber_partition_values_contains_image
from probability.discrete_pmf import DiscretePMF, is_pmf, pmf_event_prob,
    pmf_event_prob_eq_support_intersection, finite_event_indicator,
    finite_event_indicator_one_of_contains, finite_event_indicator_zero_of_not_contains,
    discrete_expectation, discrete_variance, discrete_variance_formula,
    discrete_expectation_indicator_eq_event_prob, square_fn, mass_weighted_value,
    discrete_events_independent, discrete_events_independent_comm,
    discrete_pmf_mass_zero_outside, const_real_fn, discrete_expectation_monotone
from probability.bernoulli_pmf import bernoulli_support, bernoulli_mass,
    bernoulli_mass_true

numerals Nat

/// The singleton Boolean event containing `true`.
let bool_true_event: FiniteSet[Bool] = fs_insert(fs_empty[Bool](Set[Bool].empty_set), true)

/// True when `pmf` is the Bernoulli distribution with parameter `p`:
/// the two-point Boolean support with mass `p` at `true`.
define is_bernoulli_pmf(p: Real, pmf: DiscretePMF[Bool]) -> Bool {
    pmf.support = bernoulli_support and forall(b: Bool) { pmf.mass(b) = bernoulli_mass(p, b) }
}

/// The support of a Bernoulli pmf is the two-point Boolean support.
theorem is_bernoulli_pmf_support(p: Real, pmf: DiscretePMF[Bool]) {
    is_bernoulli_pmf(p, pmf) implies pmf.support = bernoulli_support
} by {
    if is_bernoulli_pmf(p, pmf) {
        is_bernoulli_pmf(p, pmf) =
            (pmf.support = bernoulli_support and forall(b: Bool) { pmf.mass(b) = bernoulli_mass(p, b) })
        pmf.support = bernoulli_support
    }
}

/// The mass of a Bernoulli pmf at each point is the Bernoulli mass.
theorem is_bernoulli_pmf_mass(p: Real, pmf: DiscretePMF[Bool]) {
    is_bernoulli_pmf(p, pmf) implies forall(b: Bool) { pmf.mass(b) = bernoulli_mass(p, b) }
} by {
    if is_bernoulli_pmf(p, pmf) {
        is_bernoulli_pmf(p, pmf) =
            (pmf.support = bernoulli_support and forall(b: Bool) { pmf.mass(b) = bernoulli_mass(p, b) })
        forall(b: Bool) { pmf.mass(b) = bernoulli_mass(p, b) }
    }
}

/// The mass of a Bernoulli pmf at `true` is the parameter `p`.
theorem is_bernoulli_pmf_mass_true(p: Real, pmf: DiscretePMF[Bool]) {
    is_bernoulli_pmf(p, pmf) implies pmf.mass(true) = p
} by {
    if is_bernoulli_pmf(p, pmf) {
        is_bernoulli_pmf_mass(p, pmf)
        forall(b: Bool) { pmf.mass(b) = bernoulli_mass(p, b) }
        pmf.mass(true) = bernoulli_mass(p, true)
        bernoulli_mass_true(p)
        bernoulli_mass(p, true) = p
        pmf.mass(true) = p
    }
}

/// The empty Boolean finite set does not contain `true`.
theorem fs_empty_bool_no_true {
    not fs_empty[Bool](Set[Bool].empty_set).contains(true)
} by {
    fs_empty[Bool](Set[Bool].empty_set).underlying_set = Set[Bool].empty_set
    empty_set_contains_eq[Bool](true)
}

/// The singleton event `{true}` contains `true`.
theorem bool_true_event_contains_true {
    bool_true_event.contains(true)
} by {
    let e: FiniteSet[Bool] = fs_empty[Bool](Set[Bool].empty_set)
    fs_insert(e, true).underlying_set = e.underlying_set.insert(true)
    insert_contains_eq(e.underlying_set, true, true)
}

/// The singleton event `{true}` omits `false`.
theorem bool_true_event_not_contains_false {
    not bool_true_event.contains(false)
} by {
    let e: FiniteSet[Bool] = fs_empty[Bool](Set[Bool].empty_set)
    e.underlying_set = Set[Bool].empty_set
    fs_insert(e, true).underlying_set = e.underlying_set.insert(true)
    insert_contains_eq(Set[Bool].empty_set, true, false)
    empty_set_contains_eq[Bool](false)
    false != true
}

/// The probability of the event `{true}` under a Bernoulli pmf is the parameter `p`.
theorem bernoulli_true_event_prob(p: Real, pmf: DiscretePMF[Bool]) {
    is_bernoulli_pmf(p, pmf) implies pmf_event_prob(pmf, bool_true_event) = p
} by {
    if is_bernoulli_pmf(p, pmf) {
        pmf_event_prob(pmf, bool_true_event) = finite_set_sum(bool_true_event, pmf.mass)
        fs_empty_bool_no_true
        finite_set_sum_insert[Bool, Real](fs_empty[Bool](Set[Bool].empty_set), true, pmf.mass)
        finite_set_sum(bool_true_event, pmf.mass) =
            pmf.mass(true) + finite_set_sum(fs_empty[Bool](Set[Bool].empty_set), pmf.mass)
        finite_set_sum_empty[Bool, Real](pmf.mass)
        finite_set_sum(fs_empty[Bool](Set[Bool].empty_set), pmf.mass) = Real.0
        is_bernoulli_pmf_mass_true(p, pmf)
        pmf.mass(true) = p
        pmf.mass(true) + Real.0 = p
        pmf_event_prob(pmf, bool_true_event) = p
    }
}

/// The expectation of the indicator of `{true}` under a Bernoulli pmf is the parameter `p`.
theorem bernoulli_indicator_expectation(p: Real, pmf: DiscretePMF[Bool]) {
    is_bernoulli_pmf(p, pmf) implies
        discrete_expectation(pmf, finite_event_indicator(bool_true_event)) = p
} by {
    if is_bernoulli_pmf(p, pmf) {
        discrete_expectation_indicator_eq_event_prob[Bool](pmf, bool_true_event)
        discrete_expectation(pmf, finite_event_indicator(bool_true_event)) =
            pmf_event_prob(pmf, bool_true_event)
        bernoulli_true_event_prob(p, pmf)
        pmf_event_prob(pmf, bool_true_event) = p
        discrete_expectation(pmf, finite_event_indicator(bool_true_event)) = p
    }
}

/// The indicator of `{true}` is idempotent pointwise: its square equals itself.
theorem bernoulli_indicator_square_fn(b: Bool) {
    square_fn(finite_event_indicator(bool_true_event), b) =
        finite_event_indicator(bool_true_event, b)
} by {
    if b {
        bool_true_event_contains_true
        finite_event_indicator_one_of_contains(bool_true_event, b)
        finite_event_indicator(bool_true_event, b) = Real.1
        square_fn(finite_event_indicator(bool_true_event), b) = Real.1 * Real.1
        Real.1 * Real.1 = Real.1
        square_fn(finite_event_indicator(bool_true_event), b) =
            finite_event_indicator(bool_true_event, b)
    }
    if not b {
        bool_true_event_not_contains_false
        not bool_true_event.contains(b)
        finite_event_indicator_zero_of_not_contains(bool_true_event, b)
        finite_event_indicator(bool_true_event, b) = Real.0
        square_fn(finite_event_indicator(bool_true_event), b) = Real.0 * Real.0
        Real.0 * Real.0 = Real.0
        square_fn(finite_event_indicator(bool_true_event), b) =
            finite_event_indicator(bool_true_event, b)
    }
    b or not b
}

/// The indicator of `{true}` is idempotent as a function: its square equals itself.
theorem bernoulli_indicator_square_fn_eq_self {
    square_fn(finite_event_indicator(bool_true_event)) = finite_event_indicator(bool_true_event)
} by {
    forall(b: Bool) {
        bernoulli_indicator_square_fn(b)
    }
    function_extensionality[Bool, Real](square_fn(finite_event_indicator(bool_true_event)),
        finite_event_indicator(bool_true_event))
}

/// The expectation of the squared indicator of `{true}` under a Bernoulli pmf is `p`.
theorem bernoulli_indicator_expectation_square(p: Real, pmf: DiscretePMF[Bool]) {
    is_bernoulli_pmf(p, pmf) implies
        discrete_expectation(pmf, square_fn(finite_event_indicator(bool_true_event))) = p
} by {
    if is_bernoulli_pmf(p, pmf) {
        bernoulli_indicator_square_fn_eq_self
        square_fn(finite_event_indicator(bool_true_event)) = finite_event_indicator(bool_true_event)
        forall(t: Bool) {
            if pmf.support.contains(t) {
                mass_weighted_value(pmf.mass, square_fn(finite_event_indicator(bool_true_event)), t) =
                    pmf.mass(t) * square_fn(finite_event_indicator(bool_true_event), t)
                square_fn(finite_event_indicator(bool_true_event), t) =
                    finite_event_indicator(bool_true_event, t)
                pmf.mass(t) * finite_event_indicator(bool_true_event, t) =
                    mass_weighted_value(pmf.mass, finite_event_indicator(bool_true_event), t)
                mass_weighted_value(pmf.mass, square_fn(finite_event_indicator(bool_true_event)), t) =
                    mass_weighted_value(pmf.mass, finite_event_indicator(bool_true_event), t)
            }
        }
        finite_set_sum_eq_of_pointwise_on_set[Bool, Real](pmf.support,
            mass_weighted_value(pmf.mass, square_fn(finite_event_indicator(bool_true_event))),
            mass_weighted_value(pmf.mass, finite_event_indicator(bool_true_event)))
        finite_set_sum(pmf.support,
            mass_weighted_value(pmf.mass, square_fn(finite_event_indicator(bool_true_event)))) =
            finite_set_sum(pmf.support,
                mass_weighted_value(pmf.mass, finite_event_indicator(bool_true_event)))
        discrete_expectation(pmf, square_fn(finite_event_indicator(bool_true_event))) =
            discrete_expectation(pmf, finite_event_indicator(bool_true_event))
        bernoulli_indicator_expectation(p, pmf)
        discrete_expectation(pmf, finite_event_indicator(bool_true_event)) = p
        discrete_expectation(pmf, square_fn(finite_event_indicator(bool_true_event))) = p
    }
}

/// The variance of the indicator of `{true}` under a Bernoulli(p) pmf is `p * (1 - p)`.
theorem bernoulli_indicator_variance(p: Real, pmf: DiscretePMF[Bool]) {
    is_bernoulli_pmf(p, pmf) implies
        discrete_variance(pmf, finite_event_indicator(bool_true_event)) = p * (Real.1 - p)
} by {
    if is_bernoulli_pmf(p, pmf) {
        discrete_variance_formula[Bool](pmf, finite_event_indicator(bool_true_event))
        discrete_variance(pmf, finite_event_indicator(bool_true_event)) =
            discrete_expectation(pmf, square_fn(finite_event_indicator(bool_true_event))) - discrete_expectation(pmf, finite_event_indicator(bool_true_event)) * discrete_expectation(pmf, finite_event_indicator(bool_true_event))
        bernoulli_indicator_expectation_square(p, pmf)
        discrete_expectation(pmf, square_fn(finite_event_indicator(bool_true_event))) = p
        bernoulli_indicator_expectation(p, pmf)
        discrete_expectation(pmf, finite_event_indicator(bool_true_event)) = p
        discrete_variance(pmf, finite_event_indicator(bool_true_event)) = p - p * p
        p - p * p = p * (Real.1 - p)
        discrete_variance(pmf, finite_event_indicator(bool_true_event)) = p * (Real.1 - p)
    }
}

/// The predicate of membership in the value event of a random variable.
define rv_value_event_contains[T, A](rv: T -> A, a: A, x: T) -> Bool {
    rv(x) = a
}

/// The value event of a random variable at `a`: the elements of the support
/// where the variable takes the value `a`.
define rv_value_event[T, A](pmf: DiscretePMF[T], rv: T -> A, a: A) -> FiniteSet[T] {
    finite_set_filter(pmf.support, rv_value_event_contains(rv, a))
}

/// Membership in a value event is support membership together with the value equality.
theorem rv_value_event_contains_eq[T, A](pmf: DiscretePMF[T], rv: T -> A, a: A, x: T) {
    rv_value_event(pmf, rv, a).contains(x) = (pmf.support.contains(x) and rv(x) = a)
} by {
    rv_value_event_contains(rv, a, x) = (rv(x) = a)
    finite_set_filter_contains_eq(pmf.support, rv_value_event_contains(rv, a), x)
    finite_set_filter(pmf.support, rv_value_event_contains(rv, a)).contains(x) =
        (pmf.support.contains(x) and rv_value_event_contains(rv, a, x))
    finite_set_filter(pmf.support, rv_value_event_contains(rv, a)).contains(x) =
        (pmf.support.contains(x) and rv(x) = a)
    rv_value_event(pmf, rv, a) = finite_set_filter(pmf.support, rv_value_event_contains(rv, a))
    rv_value_event(pmf, rv, a).contains(x) = (pmf.support.contains(x) and rv(x) = a)
}

/// A value event is contained in the pmf support.
theorem rv_value_event_subset_support[T, A](pmf: DiscretePMF[T], rv: T -> A, a: A) {
    rv_value_event(pmf, rv, a).subset_eq(pmf.support)
} by {
    finite_set_filter_subset(pmf.support, rv_value_event_contains(rv, a))
    rv_value_event(pmf, rv, a) = finite_set_filter(pmf.support, rv_value_event_contains(rv, a))
    rv_value_event(pmf, rv, a).subset_eq(pmf.support)
}

/// Two discrete random variables are independent when every pair of value events
/// is independent under the pmf.
define discrete_rv_independent[T, A, B](pmf: DiscretePMF[T], x: T -> A, y: T -> B) -> Bool {
    forall(a: A, b: B) {
        discrete_events_independent(pmf, rv_value_event(pmf, x, a), rv_value_event(pmf, y, b))
    }
}

/// Independence of two discrete random variables is symmetric.
theorem discrete_rv_independent_comm[T, A, B](pmf: DiscretePMF[T], x: T -> A, y: T -> B) {
    discrete_rv_independent(pmf, x, y) implies discrete_rv_independent(pmf, y, x)
} by {
    if discrete_rv_independent(pmf, x, y) {
        discrete_rv_independent(pmf, x, y) = forall(a: A, b: B) {
            discrete_events_independent(pmf, rv_value_event(pmf, x, a), rv_value_event(pmf, y, b))
        }
        forall(b: B, a: A) {
            discrete_events_independent_comm(pmf, rv_value_event(pmf, x, a), rv_value_event(pmf, y, b))
            discrete_events_independent(pmf, rv_value_event(pmf, y, b), rv_value_event(pmf, x, a))
        }
        discrete_rv_independent(pmf, y, x) = forall(b: B, a: A) {
            discrete_events_independent(pmf, rv_value_event(pmf, y, b), rv_value_event(pmf, x, a))
        }
        discrete_rv_independent(pmf, y, x)
    }
}

/// An event indicator is one exactly on the event.
lemma indicator_eq_one_iff_contains[T](a: FiniteSet[T], x: T) {
    (finite_event_indicator(a, x) = Real.1) = a.contains(x)
} by {
    if a.contains(x) {
        finite_event_indicator_one_of_contains(a, x)
        finite_event_indicator(a, x) = Real.1
        (finite_event_indicator(a, x) = Real.1) = a.contains(x)
    }
    if not a.contains(x) {
        finite_event_indicator_zero_of_not_contains(a, x)
        finite_event_indicator(a, x) = Real.0
        Real.0 != Real.1
        (finite_event_indicator(a, x) = Real.1) = a.contains(x)
    }
    a.contains(x) or not a.contains(x)
}

/// An event indicator is zero exactly off the event.
lemma indicator_eq_zero_iff_not_contains[T](a: FiniteSet[T], x: T) {
    (finite_event_indicator(a, x) = Real.0) = not a.contains(x)
} by {
    if a.contains(x) {
        finite_event_indicator_one_of_contains(a, x)
        finite_event_indicator(a, x) = Real.1
        Real.1 != Real.0
        (finite_event_indicator(a, x) = Real.0) = not a.contains(x)
    }
    if not a.contains(x) {
        finite_event_indicator_zero_of_not_contains(a, x)
        finite_event_indicator(a, x) = Real.0
        (finite_event_indicator(a, x) = Real.0) = not a.contains(x)
    }
    a.contains(x) or not a.contains(x)
}

/// The value event of an event indicator at one is the support restricted to the event.
theorem rv_value_event_indicator_one[T](pmf: DiscretePMF[T], a: FiniteSet[T]) {
    rv_value_event(pmf, finite_event_indicator(a), Real.1) = fs_intersection(pmf.support, a)
} by {
    forall(x: T) {
        rv_value_event_contains_eq(pmf, finite_event_indicator(a), Real.1, x)
        indicator_eq_one_iff_contains(a, x)
        finite_set_intersection_contains_eq(pmf.support, a, x)
        rv_value_event(pmf, finite_event_indicator(a), Real.1).contains(x) =
            fs_intersection(pmf.support, a).contains(x)
        rv_value_event(pmf, finite_event_indicator(a), Real.1).underlying_set.contains(x) =
            fs_intersection(pmf.support, a).underlying_set.contains(x)
    }
    set_ext(rv_value_event(pmf, finite_event_indicator(a), Real.1).underlying_set,
        fs_intersection(pmf.support, a).underlying_set)
    finite_set_eq_of_underlying_set_eq(rv_value_event(pmf, finite_event_indicator(a), Real.1),
        fs_intersection(pmf.support, a))
    rv_value_event(pmf, finite_event_indicator(a), Real.1) = fs_intersection(pmf.support, a)
}

/// The value event of an event indicator at zero is the support minus the event.
theorem rv_value_event_indicator_zero[T](pmf: DiscretePMF[T], a: FiniteSet[T]) {
    rv_value_event(pmf, finite_event_indicator(a), Real.0) = fs_difference(pmf.support, a)
} by {
    forall(x: T) {
        rv_value_event_contains_eq(pmf, finite_event_indicator(a), Real.0, x)
        indicator_eq_zero_iff_not_contains(a, x)
        finite_set_difference_contains_eq(pmf.support, a, x)
        rv_value_event(pmf, finite_event_indicator(a), Real.0).contains(x) =
            fs_difference(pmf.support, a).contains(x)
        rv_value_event(pmf, finite_event_indicator(a), Real.0).underlying_set.contains(x) =
            fs_difference(pmf.support, a).underlying_set.contains(x)
    }
    set_ext(rv_value_event(pmf, finite_event_indicator(a), Real.0).underlying_set,
        fs_difference(pmf.support, a).underlying_set)
    finite_set_eq_of_underlying_set_eq(rv_value_event(pmf, finite_event_indicator(a), Real.0),
        fs_difference(pmf.support, a))
    rv_value_event(pmf, finite_event_indicator(a), Real.0) = fs_difference(pmf.support, a)
}

/// The intersection of two finite intersections over a common set.
lemma finite_set_intersection_support_pair[T](s: FiniteSet[T], a: FiniteSet[T], b: FiniteSet[T]) {
    fs_intersection(fs_intersection(s, a), fs_intersection(s, b)) =
        fs_intersection(s, fs_intersection(a, b))
} by {
    finite_set_intersection_assoc(s, a, fs_intersection(s, b))
    fs_intersection(fs_intersection(s, a), fs_intersection(s, b)) =
        fs_intersection(s, fs_intersection(a, fs_intersection(s, b)))
    finite_set_intersection_assoc(a, s, b)
    fs_intersection(a, fs_intersection(s, b)) = fs_intersection(fs_intersection(a, s), b)
    finite_set_intersection_comm(a, s)
    fs_intersection(a, s) = fs_intersection(s, a)
    fs_intersection(fs_intersection(a, s), b) = fs_intersection(fs_intersection(s, a), b)
    fs_intersection(a, fs_intersection(s, b)) = fs_intersection(fs_intersection(s, a), b)
    fs_intersection(s, fs_intersection(a, fs_intersection(s, b))) =
        fs_intersection(s, fs_intersection(fs_intersection(s, a), b))
    finite_set_intersection_assoc(s, a, b)
    fs_intersection(fs_intersection(s, a), b) = fs_intersection(s, fs_intersection(a, b))
    fs_intersection(s, fs_intersection(fs_intersection(s, a), b)) =
        fs_intersection(s, fs_intersection(s, fs_intersection(a, b)))
    finite_set_intersection_assoc(s, s, fs_intersection(a, b))
    fs_intersection(fs_intersection(s, s), fs_intersection(a, b)) =
        fs_intersection(s, fs_intersection(s, fs_intersection(a, b)))
    fs_intersection(s, fs_intersection(s, fs_intersection(a, b))) =
        fs_intersection(fs_intersection(s, s), fs_intersection(a, b))
    finite_set_intersection_idemp(s)
    fs_intersection(s, s) = s
    fs_intersection(fs_intersection(s, s), fs_intersection(a, b)) =
        fs_intersection(s, fs_intersection(a, b))
    fs_intersection(s, fs_intersection(a, fs_intersection(s, b))) =
        fs_intersection(s, fs_intersection(s, fs_intersection(a, b)))
    fs_intersection(fs_intersection(s, a), fs_intersection(s, b)) =
        fs_intersection(s, fs_intersection(s, fs_intersection(a, b)))
    fs_intersection(fs_intersection(s, a), fs_intersection(s, b)) =
        fs_intersection(s, fs_intersection(a, b))
}

/// If the indicators of two events are independent random variables, then the
/// events are independent: the random-variable notion extends the event notion.
theorem discrete_rv_independent_indicator_events_imp_event_independent[T](pmf: DiscretePMF[T],
    a: FiniteSet[T], b: FiniteSet[T]) {
    discrete_rv_independent(pmf, finite_event_indicator(a), finite_event_indicator(b)) implies
        discrete_events_independent(pmf, a, b)
} by {
    if discrete_rv_independent(pmf, finite_event_indicator(a), finite_event_indicator(b)) {
        discrete_rv_independent(pmf, finite_event_indicator(a), finite_event_indicator(b)) =
            forall(r: Real, s: Real) {
                discrete_events_independent(pmf, rv_value_event(pmf, finite_event_indicator(a), r),
                    rv_value_event(pmf, finite_event_indicator(b), s))
            }
        discrete_events_independent(pmf, rv_value_event(pmf, finite_event_indicator(a), Real.1),
            rv_value_event(pmf, finite_event_indicator(b), Real.1))
        rv_value_event_indicator_one(pmf, a)
        rv_value_event_indicator_one(pmf, b)
        discrete_events_independent(pmf, fs_intersection(pmf.support, a),
            fs_intersection(pmf.support, b))
        discrete_events_independent(pmf, fs_intersection(pmf.support, a),
            fs_intersection(pmf.support, b)) =
            (pmf_event_prob(pmf, fs_intersection(fs_intersection(pmf.support, a),
                fs_intersection(pmf.support, b))) =
                pmf_event_prob(pmf, fs_intersection(pmf.support, a)) * pmf_event_prob(pmf, fs_intersection(pmf.support, b)))
        pmf_event_prob(pmf, fs_intersection(fs_intersection(pmf.support, a),
            fs_intersection(pmf.support, b))) =
            pmf_event_prob(pmf, fs_intersection(pmf.support, a)) * pmf_event_prob(pmf, fs_intersection(pmf.support, b))
        finite_set_intersection_support_pair(pmf.support, a, b)
        fs_intersection(fs_intersection(pmf.support, a), fs_intersection(pmf.support, b)) =
            fs_intersection(pmf.support, fs_intersection(a, b))
        pmf_event_prob(pmf, fs_intersection(fs_intersection(pmf.support, a),
            fs_intersection(pmf.support, b))) =
            pmf_event_prob(pmf, fs_intersection(pmf.support, fs_intersection(a, b)))
        pmf_event_prob_eq_support_intersection(pmf, fs_intersection(a, b))
        pmf_event_prob(pmf, fs_intersection(a, b)) =
            pmf_event_prob(pmf, fs_intersection(pmf.support, fs_intersection(a, b)))
        pmf_event_prob(pmf, fs_intersection(pmf.support, fs_intersection(a, b))) =
            pmf_event_prob(pmf, fs_intersection(a, b))
        pmf_event_prob_eq_support_intersection(pmf, a)
        pmf_event_prob(pmf, a) = pmf_event_prob(pmf, fs_intersection(pmf.support, a))
        pmf_event_prob(pmf, fs_intersection(pmf.support, a)) = pmf_event_prob(pmf, a)
        pmf_event_prob_eq_support_intersection(pmf, b)
        pmf_event_prob(pmf, b) = pmf_event_prob(pmf, fs_intersection(pmf.support, b))
        pmf_event_prob(pmf, fs_intersection(pmf.support, b)) = pmf_event_prob(pmf, b)
        pmf_event_prob(pmf, fs_intersection(a, b)) =
            pmf_event_prob(pmf, a) * pmf_event_prob(pmf, b)
        discrete_events_independent(pmf, a, b) =
            (pmf_event_prob(pmf, fs_intersection(a, b)) =
                pmf_event_prob(pmf, a) * pmf_event_prob(pmf, b))
        discrete_events_independent(pmf, a, b)
    }
}

/// A unique consed list does not contain its head in its tail.
lemma cons_unique_not_tail_contains[A](head: A, tail: List[A]) {
    List.cons[A](head, tail).is_unique implies not tail.contains(head)
} by {
    if List.cons[A](head, tail).is_unique {
        if tail.contains(head) {
            list_contains_implies_count_geq_one[A](tail, head)
            tail.count(head) >= Nat.1
            List.cons[A](head, tail).count(head) = Nat.1 + tail.count(head)
            sum_lte(Nat.1, Nat.1, Nat.1, tail.count(head))
            Nat.1 + Nat.1 <= Nat.1 + tail.count(head)
            Nat.1 + Nat.1 = Nat.1.suc
            Nat.1.suc <= Nat.1 + tail.count(head)
            List.cons[A](head, tail).count(head) >= Nat.1.suc
            lt_suc(Nat.1)
            Nat.1 < Nat.1.suc
            lt_and_lte(Nat.1, Nat.1.suc, List.cons[A](head, tail).count(head))
            Nat.1 < List.cons[A](head, tail).count(head)
            unique_implies_no_duplicate[A](List.cons[A](head, tail), head)
            List.cons[A](head, tail).count(head) <= Nat.1
            lte_imp_not_lt(List.cons[A](head, tail).count(head), Nat.1)
            not (Nat.1 < List.cons[A](head, tail).count(head))
            false
        }
        not tail.contains(head)
    }
}

/// The indicator that a random variable takes the value `a` at `t`.
define rv_value_indicator[T](rv: T -> Real, t: T, a: Real) -> Real {
    if rv(t) = a { Real.1 } else { Real.0 }
}

/// A value indicator is one when the value is attained.
theorem rv_value_indicator_one_of_eq[T](rv: T -> Real, t: T, a: Real) {
    rv(t) = a implies rv_value_indicator(rv, t, a) = Real.1
} by {
    if rv(t) = a {
        rv_value_indicator(rv, t, a) = Real.1
    }
}

/// A value indicator is zero when the value is not attained.
theorem rv_value_indicator_zero_of_not_eq[T](rv: T -> Real, t: T, a: Real) {
    rv(t) != a implies rv_value_indicator(rv, t, a) = Real.0
} by {
    if rv(t) != a {
        if rv(t) = a {
            false
        }
        rv_value_indicator(rv, t, a) = Real.0
    }
}

/// The `a`-weighted value indicator at `t`.
define rv_expansion_term[T](rv: T -> Real, t: T, a: Real) -> Real {
    a * rv_value_indicator(rv, t, a)
}

/// A random variable expands over its image values: at a support point, the sum of
/// the value-weighted indicators over the canonical image list is the value itself.
theorem rv_expansion_at_point[T](pmf: DiscretePMF[T], rv: T -> Real, t: T) {
    pmf.support.contains(t) implies
        sum[Real](map(finite_fiber_partition_values(pmf.support, rv), rv_expansion_term(rv, t))) = rv(t)
} by {
    if pmf.support.contains(t) {
        let values = finite_fiber_partition_values(pmf.support, rv)
        define p(l: List[Real]) -> Bool {
            l.is_unique implies sum[Real](map(l, rv_expansion_term(rv, t))) =
                if l.contains(rv(t)) { rv(t) } else { Real.0 }
        }

        map(List.nil[Real], rv_expansion_term(rv, t)) = List.nil[Real]
        sum[Real](List.nil[Real]) = Real.0
        List.nil[Real].is_unique
        List.nil[Real].contains(rv(t)) = false
        sum[Real](map(List.nil[Real], rv_expansion_term(rv, t))) =
            if List.nil[Real].contains(rv(t)) { rv(t) } else { Real.0 }
        p(List.nil[Real])

        forall(head: Real, tail: List[Real]) {
            if p(tail) {
                if List.cons(head, tail).is_unique {
                    unique_implies_tail_unique[Real](head, tail)
                    tail.is_unique
                    p(tail) = (tail.is_unique implies sum[Real](map(tail, rv_expansion_term(rv, t))) =
                        if tail.contains(rv(t)) { rv(t) } else { Real.0 })
                    sum[Real](map(tail, rv_expansion_term(rv, t))) =
                        if tail.contains(rv(t)) { rv(t) } else { Real.0 }
                    map(List.cons(head, tail), rv_expansion_term(rv, t)) =
                        List.cons(rv_expansion_term(rv, t, head), map(tail, rv_expansion_term(rv, t)))
                    sum[Real](List.cons(rv_expansion_term(rv, t, head), map(tail, rv_expansion_term(rv, t)))) =
                        rv_expansion_term(rv, t, head) + sum[Real](map(tail, rv_expansion_term(rv, t)))
                    sum[Real](map(List.cons(head, tail), rv_expansion_term(rv, t))) =
                        rv_expansion_term(rv, t, head) + sum[Real](map(tail, rv_expansion_term(rv, t)))
                    cons_unique_not_tail_contains[Real](head, tail)
                    not tail.contains(head)
                    if head = rv(t) {
                        rv_value_indicator_one_of_eq(rv, t, head)
                        rv_value_indicator(rv, t, head) = Real.1
                        rv_expansion_term(rv, t, head) = head * Real.1
                        head * Real.1 = head
                        rv_expansion_term(rv, t, head) = head
                        rv_expansion_term(rv, t, head) = rv(t)
                        tail.contains(head) = false
                        tail.contains(rv(t)) = false
                        if tail.contains(rv(t)) {
                            false
                        }
                        sum[Real](map(tail, rv_expansion_term(rv, t))) = Real.0
                        sum[Real](map(List.cons(head, tail), rv_expansion_term(rv, t))) =
                            rv(t) + Real.0
                        rv(t) + Real.0 = rv(t)
                        List.cons(head, tail).contains(rv(t)) = true
                        sum[Real](map(List.cons(head, tail), rv_expansion_term(rv, t))) =
                            if List.cons(head, tail).contains(rv(t)) { rv(t) } else { Real.0 }
                    }
                    if head != rv(t) {
                        rv_value_indicator_zero_of_not_eq(rv, t, head)
                        rv_value_indicator(rv, t, head) = Real.0
                        rv_expansion_term(rv, t, head) = head * Real.0
                        head * Real.0 = Real.0
                        rv_expansion_term(rv, t, head) = Real.0
                        if tail.contains(rv(t)) {
                            sum[Real](map(tail, rv_expansion_term(rv, t))) = rv(t)
                            sum[Real](map(List.cons(head, tail), rv_expansion_term(rv, t))) = Real.0 + rv(t)
                            Real.0 + rv(t) = rv(t)
                            List.cons(head, tail).contains(rv(t)) = true
                            sum[Real](map(List.cons(head, tail), rv_expansion_term(rv, t))) =
                                if List.cons(head, tail).contains(rv(t)) { rv(t) } else { Real.0 }
                        }
                        if not tail.contains(rv(t)) {
                            sum[Real](map(tail, rv_expansion_term(rv, t))) = Real.0
                            sum[Real](map(List.cons(head, tail), rv_expansion_term(rv, t))) = Real.0 + Real.0
                            Real.0 + Real.0 = Real.0
                            List.cons(head, tail).contains(rv(t)) = false
                            sum[Real](map(List.cons(head, tail), rv_expansion_term(rv, t))) =
                                if List.cons(head, tail).contains(rv(t)) { rv(t) } else { Real.0 }
                        }
                        tail.contains(rv(t)) or not tail.contains(rv(t))
                        sum[Real](map(List.cons(head, tail), rv_expansion_term(rv, t))) =
                            if List.cons(head, tail).contains(rv(t)) { rv(t) } else { Real.0 }
                    }
                    head = rv(t) or head != rv(t)
                    sum[Real](map(List.cons(head, tail), rv_expansion_term(rv, t))) =
                        if List.cons(head, tail).contains(rv(t)) { rv(t) } else { Real.0 }
                }
                p(List.cons(head, tail))
            }
        }

        List.induction(function(l: List[Real]) { p(l) })
        function[T0](x0: List[T0] -> Bool, x1: List[T0]) {
            not x0(List.nil[T0]) or exists(k0: T0, k1: List[T0]) {
                x0(k1) and not x0(List.cons[T0](k0, k1))
            } or x0(x1)
        }[Real](p, values)
        p(values)
        function(x0: List[Real]) {
            (x0.is_unique implies sum[Real](map(x0, rv_expansion_term(rv, t))) =
                if x0.contains(rv(t)) { rv(t) } else { Real.0 }) = p(x0)
        }(values)
        values.is_unique implies sum[Real](map(values, rv_expansion_term(rv, t))) =
            if values.contains(rv(t)) { rv(t) } else { Real.0 }
        finite_fiber_partition_values_spec(pmf.support, rv)
        values.is_unique
        sum[Real](map(values, rv_expansion_term(rv, t))) =
            if values.contains(rv(t)) { rv(t) } else { Real.0 }
        finite_fiber_partition_values_contains_image(pmf.support, rv, t)
        values.contains(rv(t))
        if values.contains(rv(t)) {
            sum[Real](map(values, rv_expansion_term(rv, t))) = rv(t)
        }
        if not values.contains(rv(t)) {
            false
        }
        values.contains(rv(t)) or not values.contains(rv(t))
        sum[Real](map(values, rv_expansion_term(rv, t))) = rv(t)
        sum[Real](map(finite_fiber_partition_values(pmf.support, rv), rv_expansion_term(rv, t))) = rv(t)
    }
}

/// Scalar multiplication distributes over a list sum of mapped values.
lemma list_sum_scalar_mul[U](c: Real, items: List[U], f: U -> Real) {
    c * sum[Real](map(items, f)) = sum[Real](map(items, mul_fn(c, f)))
} by {
    sum_scalar_mul[Real](c, map(items, f))
    c * sum[Real](map(items, f)) = sum[Real](map(map(items, f), scalar_mul[Real](c)))
    map_map[U, Real, Real](items, f, scalar_mul[Real](c))
    map(map(items, f), scalar_mul[Real](c)) = map(items, compose[U, Real, Real](scalar_mul[Real](c), f))
    forall(a: U) {
        compose[U, Real, Real](scalar_mul[Real](c), f, a) = scalar_mul[Real](c, f(a))
        scalar_mul[Real](c, f(a)) = c * f(a)
        mul_fn(c, f, a) = c * f(a)
        compose[U, Real, Real](scalar_mul[Real](c), f, a) = mul_fn(c, f, a)
    }
    function_extensionality[U, Real](compose[U, Real, Real](scalar_mul[Real](c), f), mul_fn(c, f))
    compose[U, Real, Real](scalar_mul[Real](c), f) = mul_fn(c, f)
    map(items, compose[U, Real, Real](scalar_mul[Real](c), f)) = map(items, mul_fn(c, f))
    sum[Real](map(map(items, f), scalar_mul[Real](c))) = sum[Real](map(items, mul_fn(c, f)))
    c * sum[Real](map(items, f)) = sum[Real](map(items, mul_fn(c, f)))
}

/// Mapping over a consed list is consing the mapped head onto the mapped tail.
lemma map_cons_general[U, R](head: U, tail: List[U], f: U -> R) {
    map(List.cons(head, tail), f) = List.cons(f(head), map(tail, f))
}

/// The list sum of a two-argument function over its first argument.
define list_sum_fn[T, U](items: List[U], g: U -> T -> Real, t: T) -> Real {
    sum[Real](map(items, function(a: U) { g(a, t) }))
}

/// The finite-set sum of a two-argument function over its second argument.
define finite_set_sum_fn[T, U](s: FiniteSet[T], g: U -> T -> Real, a: U) -> Real {
    finite_set_sum(s, g(a))
}

/// The finite-set sum of a zero-valued function is zero.
lemma finite_set_sum_const_zero[T](s: FiniteSet[T]) {
    finite_set_sum(s, const_real_fn[T](Real.0)) = Real.0
} by {
    let one_fn = const_real_fn[T](Real.1)
    let zero_scaled = mul_fn(Real.0, one_fn)
    finite_set_sum_scalar_mul[T, Real](Real.0, s, one_fn)
    Real.0 * finite_set_sum(s, one_fn) = finite_set_sum(s, zero_scaled)
    Real.0 * finite_set_sum(s, one_fn) = Real.0
    finite_set_sum(s, zero_scaled) = Real.0
    forall(x: T) {
        zero_scaled(x) = Real.0 * one_fn(x)
        zero_scaled(x) = const_real_fn[T](Real.0, x)
    }
    function_extensionality[T, Real](zero_scaled, const_real_fn[T](Real.0))
    finite_set_sum(s, zero_scaled) = finite_set_sum(s, const_real_fn[T](Real.0))
    finite_set_sum(s, const_real_fn[T](Real.0)) = Real.0
}

/// A finite-set sum commutes with a list sum: the sum over the finite set of the
/// list sums equals the list of the finite-set sums.
lemma finite_set_sum_list_sum_swap[T, U](s: FiniteSet[T], items: List[U], g: U -> T -> Real) {
    finite_set_sum(s, list_sum_fn(items, g)) = sum[Real](map(items, finite_set_sum_fn(s, g)))
} by {
    define p(l: List[U]) -> Bool {
        finite_set_sum(s, list_sum_fn(l, g)) = sum[Real](map(l, finite_set_sum_fn(s, g)))
    }

    forall(t: T) {
        list_sum_fn(List.nil[U], g, t) = sum[Real](map(List.nil[U], function(a: U) { g(a, t) }))
        map(List.nil[U], function(a: U) { g(a, t) }) = List.nil[Real]
        sum[Real](List.nil[Real]) = Real.0
        list_sum_fn(List.nil[U], g, t) = Real.0
        list_sum_fn(List.nil[U], g, t) = const_real_fn[T](Real.0, t)
    }
    function_extensionality[T, Real](list_sum_fn(List.nil[U], g), const_real_fn[T](Real.0))
    list_sum_fn(List.nil[U], g) = const_real_fn[T](Real.0)
    finite_set_sum(s, list_sum_fn(List.nil[U], g)) = finite_set_sum(s, const_real_fn[T](Real.0))
    finite_set_sum_const_zero(s)
    finite_set_sum(s, const_real_fn[T](Real.0)) = Real.0
    finite_set_sum(s, list_sum_fn(List.nil[U], g)) = Real.0
    map(List.nil[U], finite_set_sum_fn(s, g)) = List.nil[Real]
    sum[Real](List.nil[Real]) = Real.0
    sum[Real](map(List.nil[U], finite_set_sum_fn(s, g))) = Real.0
    p(List.nil[U])

    forall(head: U, tail: List[U]) {
        if p(tail) {
            forall(t: T) {
                list_sum_fn(List.cons(head, tail), g, t) =
                    sum[Real](map(List.cons(head, tail), function(a: U) { g(a, t) }))
                map_cons_general[U, Real](head, tail, function(a: U) { g(a, t) })
                map(List.cons(head, tail), function(a: U) { g(a, t) }) =
                    List.cons(g(head, t), map(tail, function(a: U) { g(a, t) }))
                sum[Real](List.cons(g(head, t), map(tail, function(a: U) { g(a, t) }))) =
                    g(head, t) + sum[Real](map(tail, function(a: U) { g(a, t) }))
                list_sum_fn(List.cons(head, tail), g, t) = g(head, t) + list_sum_fn(tail, g, t)
                add_fn(g(head), list_sum_fn(tail, g), t) = g(head, t) + list_sum_fn(tail, g, t)
                list_sum_fn(List.cons(head, tail), g, t) = add_fn(g(head), list_sum_fn(tail, g), t)
            }
            function_extensionality[T, Real](list_sum_fn(List.cons(head, tail), g),
                add_fn(g(head), list_sum_fn(tail, g)))
            list_sum_fn(List.cons(head, tail), g) = add_fn(g(head), list_sum_fn(tail, g))
            finite_set_sum(s, list_sum_fn(List.cons(head, tail), g)) =
                finite_set_sum(s, add_fn(g(head), list_sum_fn(tail, g)))
            finite_set_sum_add[T, Real](s, g(head), list_sum_fn(tail, g))
            finite_set_sum(s, add_fn(g(head), list_sum_fn(tail, g))) =
                finite_set_sum(s, g(head)) + finite_set_sum(s, list_sum_fn(tail, g))
            finite_set_sum(s, list_sum_fn(List.cons(head, tail), g)) =
                finite_set_sum(s, g(head)) + finite_set_sum(s, list_sum_fn(tail, g))
            p(tail) = (finite_set_sum(s, list_sum_fn(tail, g)) =
                sum[Real](map(tail, finite_set_sum_fn(s, g))))
            finite_set_sum(s, list_sum_fn(tail, g)) = sum[Real](map(tail, finite_set_sum_fn(s, g)))
            finite_set_sum(s, list_sum_fn(List.cons(head, tail), g)) =
                finite_set_sum(s, g(head)) + sum[Real](map(tail, finite_set_sum_fn(s, g)))
            map(List.cons(head, tail), finite_set_sum_fn(s, g)) =
                List.cons(finite_set_sum_fn(s, g, head), map(tail, finite_set_sum_fn(s, g)))
            finite_set_sum_fn(s, g, head) = finite_set_sum(s, g(head))
            map(List.cons(head, tail), finite_set_sum_fn(s, g)) =
                List.cons(finite_set_sum(s, g(head)), map(tail, finite_set_sum_fn(s, g)))
            sum[Real](List.cons(finite_set_sum(s, g(head)), map(tail, finite_set_sum_fn(s, g)))) =
                finite_set_sum(s, g(head)) + sum[Real](map(tail, finite_set_sum_fn(s, g)))
            sum[Real](map(List.cons(head, tail), finite_set_sum_fn(s, g))) =
                finite_set_sum(s, g(head)) + sum[Real](map(tail, finite_set_sum_fn(s, g)))
            finite_set_sum(s, list_sum_fn(List.cons(head, tail), g)) =
                sum[Real](map(List.cons(head, tail), finite_set_sum_fn(s, g)))
            p(List.cons(head, tail))
        }
    }

    List.induction(function(l: List[U]) { p(l) })
    function[T0](x0: List[T0] -> Bool, x1: List[T0]) {
        not x0(List.nil[T0]) or exists(k0: T0, k1: List[T0]) {
            x0(k1) and not x0(List.cons[T0](k0, k1))
        } or x0(x1)
    }[U](p, items)
    function(x0: List[U]) {
        (finite_set_sum(s, list_sum_fn(x0, g)) = sum[Real](map(x0, finite_set_sum_fn(s, g)))) = p(x0)
    }(items)
    finite_set_sum(s, list_sum_fn(items, g)) = sum[Real](map(items, finite_set_sum_fn(s, g)))
}

/// A subset of a finite set intersects it to itself.
lemma finite_set_intersection_eq_right_of_subset[T](left: FiniteSet[T], right: FiniteSet[T]) {
    right.subset_eq(left) implies fs_intersection(left, right) = right
} by {
    if right.subset_eq(left) {
        intersection_with_superset_is_self(right.underlying_set, left.underlying_set)
        right.underlying_set.intersection(left.underlying_set) = right.underlying_set
        intersection_comm(left.underlying_set, right.underlying_set)
        left.underlying_set.intersection(right.underlying_set) = right.underlying_set
        fs_intersection(left, right).underlying_set =
            left.underlying_set.intersection(right.underlying_set)
        fs_intersection(left, right).underlying_set = right.underlying_set
        finite_set_ext(fs_intersection(left, right), right)
        fs_intersection(left, right) = right
    }
}

/// The finite-set sum of a function vanishing on the finite set is zero.
lemma finite_set_sum_zero_of_contains[T](s: FiniteSet[T], f: T -> Real) {
    (forall(x: T) { s.contains(x) implies f(x) = Real.0 }) implies finite_set_sum(s, f) = Real.0
} by {
    if forall(x: T) { s.contains(x) implies f(x) = Real.0 } {
        finite_set_sum_eq_of_pointwise_on_set(s, f, const_real_fn[T](Real.0))
        finite_set_sum(s, f) = finite_set_sum(s, const_real_fn[T](Real.0))
        finite_set_sum_const_zero(s)
        finite_set_sum(s, const_real_fn[T](Real.0)) = Real.0
        finite_set_sum(s, f) = Real.0
    }
}

/// The mass-weighted indicator that a random variable takes the value `a` at `t`.
define mass_value_indicator[T](mass: T -> Real, rv: T -> Real, a: Real, t: T) -> Real {
    mass(t) * rv_value_indicator(rv, t, a)
}

/// The `a`-weighted mass-value indicator used to expand an expectation over values.
define mass_weighted_expansion_term[T](mass: T -> Real, rv: T -> Real, a: Real, t: T) -> Real {
    a * mass_value_indicator(mass, rv, a, t)
}

/// The value times the probability of the value event.
define rv_value_mass_term[T](pmf: DiscretePMF[T], rv: T -> Real, a: Real) -> Real {
    a * pmf_event_prob(pmf, rv_value_event(pmf, rv, a))
}

/// Summing the mass-weighted value indicator over the support gives the probability
/// of the value event.
theorem finite_set_sum_mass_value_indicator_eq_event_prob[T](pmf: DiscretePMF[T], rv: T -> Real, a: Real) {
    finite_set_sum(pmf.support, mass_value_indicator(pmf.mass, rv, a)) =
        pmf_event_prob(pmf, rv_value_event(pmf, rv, a))
} by {
    let inside = rv_value_event(pmf, rv, a)
    let outside = fs_difference(pmf.support, inside)
    let mvi = mass_value_indicator(pmf.mass, rv, a)

    finite_set_intersection_union_difference_is_self[T](pmf.support, inside)
    finite_set_intersection_is_disjoint_difference[T](pmf.support, inside)
    rv_value_event_subset_support(pmf, rv, a)
    inside.subset_eq(pmf.support)
    finite_set_intersection_eq_right_of_subset(pmf.support, inside)
    fs_intersection(pmf.support, inside) = inside
    finite_set_sum_disjoint_union[T, Real](inside, outside, mvi)
    finite_set_sum(fs_union(inside, outside), mvi) =
        finite_set_sum(inside, mvi) + finite_set_sum(outside, mvi)

    forall(x: T) {
        if inside.contains(x) {
            rv_value_event_contains_eq(pmf, rv, a, x)
            rv(x) = a
            rv_value_indicator_one_of_eq(rv, x, a)
            rv_value_indicator(rv, x, a) = Real.1
            mvi(x) = pmf.mass(x) * rv_value_indicator(rv, x, a)
            pmf.mass(x) * Real.1 = pmf.mass(x)
            mvi(x) = pmf.mass(x)
        }
    }
    forall(x: T) { inside.contains(x) implies mvi(x) = pmf.mass(x) }
    finite_set_sum_eq_of_pointwise_on_set[T, Real](inside, mvi, pmf.mass)
    finite_set_sum(inside, mvi) = finite_set_sum(inside, pmf.mass)

    forall(x: T) {
        if outside.contains(x) {
            finite_set_difference_contains_eq(pmf.support, inside, x)
            outside.contains(x) = (pmf.support.contains(x) and not inside.contains(x))
            pmf.support.contains(x)
            rv_value_event_contains_eq(pmf, rv, a, x)
            inside.contains(x) = (pmf.support.contains(x) and rv(x) = a)
            if rv(x) = a {
                inside.contains(x)
                false
            }
            rv(x) != a
            rv_value_indicator_zero_of_not_eq(rv, x, a)
            rv_value_indicator(rv, x, a) = Real.0
            mvi(x) = pmf.mass(x) * rv_value_indicator(rv, x, a)
            pmf.mass(x) * Real.0 = Real.0
            mvi(x) = Real.0
        }
    }
    forall(x: T) { outside.contains(x) implies mvi(x) = Real.0 }
    finite_set_sum_zero_of_contains(outside, mvi)
    finite_set_sum(outside, mvi) = Real.0

    finite_set_sum(inside, pmf.mass) + Real.0 = finite_set_sum(inside, pmf.mass)
    finite_set_sum(fs_union(inside, outside), mvi) =
        finite_set_sum(inside, pmf.mass) + Real.0
    finite_set_sum(fs_union(inside, outside), mvi) = finite_set_sum(inside, pmf.mass)
    fs_union(fs_intersection(pmf.support, inside), outside) = pmf.support
    fs_union(inside, outside) = pmf.support
    finite_set_sum(pmf.support, mvi) = finite_set_sum(inside, pmf.mass)
    pmf_event_prob(pmf, inside) = finite_set_sum(inside, pmf.mass)
    finite_set_sum(pmf.support, mvi) = pmf_event_prob(pmf, inside)
    finite_set_sum(pmf.support, mvi) = pmf_event_prob(pmf, rv_value_event(pmf, rv, a))
}

/// The law of the unconscious statistician: the expectation of a random variable
/// is the sum over its image values of the value times the probability of that value.
theorem discrete_expectation_eq_value_sum[T](pmf: DiscretePMF[T], rv: T -> Real) {
    discrete_expectation(pmf, rv) =
        sum[Real](map(finite_fiber_partition_values(pmf.support, rv), rv_value_mass_term(pmf, rv)))
} by {
    let values = finite_fiber_partition_values(pmf.support, rv)
    let mwet = mass_weighted_expansion_term(pmf.mass, rv)
    let mwv = mass_weighted_value(pmf.mass, rv)

    forall(t: T) {
        if pmf.support.contains(t) {
            rv_expansion_at_point(pmf, rv, t)
            sum[Real](map(values, rv_expansion_term(rv, t))) = rv(t)
            mwv(t) = pmf.mass(t) * rv(t)
            list_sum_scalar_mul[Real](pmf.mass(t), values, rv_expansion_term(rv, t))
            pmf.mass(t) * sum[Real](map(values, rv_expansion_term(rv, t))) =
                sum[Real](map(values, mul_fn(pmf.mass(t), rv_expansion_term(rv, t))))
            forall(a: Real) {
                mul_fn(pmf.mass(t), rv_expansion_term(rv, t), a) =
                    pmf.mass(t) * rv_expansion_term(rv, t, a)
                rv_expansion_term(rv, t, a) = a * rv_value_indicator(rv, t, a)
                pmf.mass(t) * rv_expansion_term(rv, t, a) =
                    pmf.mass(t) * (a * rv_value_indicator(rv, t, a))
                pmf.mass(t) * (a * rv_value_indicator(rv, t, a)) =
                    a * (pmf.mass(t) * rv_value_indicator(rv, t, a))
                mass_value_indicator(pmf.mass, rv, a, t) =
                    pmf.mass(t) * rv_value_indicator(rv, t, a)
                pmf.mass(t) * rv_expansion_term(rv, t, a) =
                    a * mass_value_indicator(pmf.mass, rv, a, t)
                mass_weighted_expansion_term(pmf.mass, rv, a, t) =
                    a * mass_value_indicator(pmf.mass, rv, a, t)
                pmf.mass(t) * rv_expansion_term(rv, t, a) =
                    mass_weighted_expansion_term(pmf.mass, rv, a, t)
                mul_fn(pmf.mass(t), rv_expansion_term(rv, t), a) =
                    mass_weighted_expansion_term(pmf.mass, rv, a, t)
                mass_weighted_expansion_term(pmf.mass, rv, a, t) = mwet(a, t)
                mul_fn(pmf.mass(t), rv_expansion_term(rv, t), a) = mwet(a, t)
            }
            function_extensionality[Real, Real](mul_fn(pmf.mass(t), rv_expansion_term(rv, t)),
                function(a: Real) { mwet(a, t) })
            mul_fn(pmf.mass(t), rv_expansion_term(rv, t)) = function(a: Real) { mwet(a, t) }
            sum_map_of_pointwise[Real, Real](values, mul_fn(pmf.mass(t), rv_expansion_term(rv, t)),
                function(a: Real) { mwet(a, t) })
            sum[Real](map(values, mul_fn(pmf.mass(t), rv_expansion_term(rv, t)))) =
                sum[Real](map(values, function(a: Real) { mwet(a, t) }))
            pmf.mass(t) * sum[Real](map(values, rv_expansion_term(rv, t))) =
                sum[Real](map(values, function(a: Real) { mwet(a, t) }))
            mwv(t) = sum[Real](map(values, function(a: Real) { mwet(a, t) }))
            list_sum_fn(values, mwet, t) =
                sum[Real](map(values, function(a: Real) { mwet(a, t) }))
            sum[Real](map(values, function(a: Real) { mwet(a, t) })) =
                list_sum_fn(values, mwet, t)
            mwv(t) = list_sum_fn(values, mwet, t)
        }
    }
    forall(t: T) { pmf.support.contains(t) implies mwv(t) = list_sum_fn(values, mwet, t) }
    finite_set_sum_eq_of_pointwise_on_set[T, Real](pmf.support, mwv, list_sum_fn(values, mwet))
    finite_set_sum(pmf.support, mwv) = finite_set_sum(pmf.support, list_sum_fn(values, mwet))
    discrete_expectation(pmf, rv) = finite_set_sum(pmf.support, mwv)
    discrete_expectation(pmf, rv) = finite_set_sum(pmf.support, list_sum_fn(values, mwet))
    finite_set_sum_list_sum_swap(pmf.support, values, mwet)
    finite_set_sum(pmf.support, list_sum_fn(values, mwet)) =
        sum[Real](map(values, finite_set_sum_fn(pmf.support, mwet)))
    discrete_expectation(pmf, rv) = sum[Real](map(values, finite_set_sum_fn(pmf.support, mwet)))

    forall(a: Real) {
        finite_set_sum_fn(pmf.support, mwet, a) = finite_set_sum(pmf.support, mwet(a))
        mwet(a) = mass_weighted_expansion_term(pmf.mass, rv, a)
        finite_set_sum(pmf.support, mwet(a)) =
            finite_set_sum(pmf.support, mass_weighted_expansion_term(pmf.mass, rv, a))
        forall(t: T) {
            mass_weighted_expansion_term(pmf.mass, rv, a, t) =
                a * mass_value_indicator(pmf.mass, rv, a, t)
            mul_fn(a, mass_value_indicator(pmf.mass, rv, a), t) =
                a * mass_value_indicator(pmf.mass, rv, a, t)
            mass_weighted_expansion_term(pmf.mass, rv, a, t) =
                mul_fn(a, mass_value_indicator(pmf.mass, rv, a), t)
        }
        function_extensionality[T, Real](mass_weighted_expansion_term(pmf.mass, rv, a),
            mul_fn(a, mass_value_indicator(pmf.mass, rv, a)))
        mass_weighted_expansion_term(pmf.mass, rv, a) =
            mul_fn(a, mass_value_indicator(pmf.mass, rv, a))
        finite_set_sum(pmf.support, mass_weighted_expansion_term(pmf.mass, rv, a)) =
            finite_set_sum(pmf.support, mul_fn(a, mass_value_indicator(pmf.mass, rv, a)))
        finite_set_sum_scalar_mul[T, Real](a, pmf.support, mass_value_indicator(pmf.mass, rv, a))
        a * finite_set_sum(pmf.support, mass_value_indicator(pmf.mass, rv, a)) =
            finite_set_sum(pmf.support, mul_fn(a, mass_value_indicator(pmf.mass, rv, a)))
        finite_set_sum(pmf.support, mass_weighted_expansion_term(pmf.mass, rv, a)) =
            a * finite_set_sum(pmf.support, mass_value_indicator(pmf.mass, rv, a))
        finite_set_sum_mass_value_indicator_eq_event_prob(pmf, rv, a)
        finite_set_sum(pmf.support, mass_value_indicator(pmf.mass, rv, a)) =
            pmf_event_prob(pmf, rv_value_event(pmf, rv, a))
        a * finite_set_sum(pmf.support, mass_value_indicator(pmf.mass, rv, a)) =
            a * pmf_event_prob(pmf, rv_value_event(pmf, rv, a))
        finite_set_sum(pmf.support, mass_weighted_expansion_term(pmf.mass, rv, a)) =
            a * pmf_event_prob(pmf, rv_value_event(pmf, rv, a))
        rv_value_mass_term(pmf, rv, a) = a * pmf_event_prob(pmf, rv_value_event(pmf, rv, a))
        finite_set_sum(pmf.support, mass_weighted_expansion_term(pmf.mass, rv, a)) =
            rv_value_mass_term(pmf, rv, a)
        finite_set_sum_fn(pmf.support, mwet, a) = rv_value_mass_term(pmf, rv, a)
    }
    forall(a: Real) {
        finite_set_sum_fn(pmf.support, mwet, a) = rv_value_mass_term(pmf, rv, a)
    }
    sum_map_of_pointwise[Real, Real](values, finite_set_sum_fn(pmf.support, mwet),
        rv_value_mass_term(pmf, rv))
    sum[Real](map(values, finite_set_sum_fn(pmf.support, mwet))) =
        sum[Real](map(values, rv_value_mass_term(pmf, rv)))
    discrete_expectation(pmf, rv) =
        sum[Real](map(values, rv_value_mass_term(pmf, rv)))
    discrete_expectation(pmf, rv) =
        sum[Real](map(finite_fiber_partition_values(pmf.support, rv), rv_value_mass_term(pmf, rv)))
}

/// The pointwise product of two real-valued random variables.
define pointwise_mul_fn[T](x: T -> Real, y: T -> Real, t: T) -> Real {
    x(t) * y(t)
}

/// The joint value indicator of two random variables: one when both values are attained.
define rv_pair_value_indicator[T](x: T -> Real, y: T -> Real, a: Real, b: Real, t: T) -> Real {
    rv_value_indicator(x, t, a) * rv_value_indicator(y, t, b)
}

/// A joint value indicator is one when both values are attained.
theorem rv_pair_value_indicator_one_of_pair[T](x: T -> Real, y: T -> Real, a: Real, b: Real, t: T) {
    x(t) = a and y(t) = b implies rv_pair_value_indicator(x, y, a, b, t) = Real.1
} by {
    if x(t) = a and y(t) = b {
        rv_value_indicator_one_of_eq(x, t, a)
        rv_value_indicator_one_of_eq(y, t, b)
        rv_value_indicator(x, t, a) = Real.1
        rv_value_indicator(y, t, b) = Real.1
        rv_pair_value_indicator(x, y, a, b, t) =
            rv_value_indicator(x, t, a) * rv_value_indicator(y, t, b)
        rv_pair_value_indicator(x, y, a, b, t) = Real.1 * Real.1
        Real.1 * Real.1 = Real.1
        rv_pair_value_indicator(x, y, a, b, t) = Real.1
    }
}

/// A joint value indicator is zero when not both values are attained.
theorem rv_pair_value_indicator_zero_of_not_pair[T](x: T -> Real, y: T -> Real, a: Real, b: Real, t: T) {
    not (x(t) = a and y(t) = b) implies rv_pair_value_indicator(x, y, a, b, t) = Real.0
} by {
    if not (x(t) = a and y(t) = b) {
        if x(t) = a {
            if y(t) = b {
                false
            }
            rv_value_indicator_one_of_eq(x, t, a)
            rv_value_indicator_zero_of_not_eq(y, t, b)
            rv_value_indicator(x, t, a) = Real.1
            rv_value_indicator(y, t, b) = Real.0
            rv_pair_value_indicator(x, y, a, b, t) =
                rv_value_indicator(x, t, a) * rv_value_indicator(y, t, b)
            rv_pair_value_indicator(x, y, a, b, t) = Real.1 * Real.0
            Real.1 * Real.0 = Real.0
            rv_pair_value_indicator(x, y, a, b, t) = Real.0
        }
        if x(t) != a {
            rv_value_indicator_zero_of_not_eq(x, t, a)
            rv_value_indicator(x, t, a) = Real.0
            rv_pair_value_indicator(x, y, a, b, t) =
                rv_value_indicator(x, t, a) * rv_value_indicator(y, t, b)
            Real.0 * rv_value_indicator(y, t, b) = Real.0
            rv_pair_value_indicator(x, y, a, b, t) = Real.0
        }
        x(t) = a or x(t) != a
        rv_pair_value_indicator(x, y, a, b, t) = Real.0
    }
}

/// The mass-weighted joint value indicator.
define mass_pair_value_indicator[T](mass: T -> Real, x: T -> Real, y: T -> Real, a: Real, b: Real, t: T) -> Real {
    mass(t) * rv_pair_value_indicator(x, y, a, b, t)
}

/// The joint value-weighted mass term used to expand a product expectation.
define mass_pair_expansion_term[T](mass: T -> Real, x: T -> Real, y: T -> Real, a: Real, b: Real, t: T) -> Real {
    a * b * mass_pair_value_indicator(mass, x, y, a, b, t)
}

/// The row sum of products with a fixed left factor.
define product_row_sum_fn2[U](right: List[U], g: U -> Real, c: Real) -> Real {
    sum[Real](map(right, mul_fn(c, g)))
}

/// The iterated sum over two lists of the products of the mapped values.
define product_double_sum_fn[T, U](left: List[T], right: List[U], f: T -> Real, g: U -> Real) -> Real {
    sum[Real](map(left, function(a: T) { product_row_sum_fn2(right, g, f(a)) }))
}

/// The product of two list sums is the iterated sum of the products.
lemma list_sum_product_of_sums[T, U](left: List[T], right: List[U], f: T -> Real, g: U -> Real) {
    sum[Real](map(left, f)) * sum[Real](map(right, g)) = product_double_sum_fn(left, right, f, g)
} by {
    define p(l: List[T]) -> Bool {
        sum[Real](map(l, f)) * sum[Real](map(right, g)) = product_double_sum_fn(l, right, f, g)
    }

    map(List.nil[T], f) = List.nil[Real]
    sum[Real](List.nil[Real]) = Real.0
    Real.0 * sum[Real](map(right, g)) = Real.0
    product_double_sum_fn(List.nil[T], right, f, g) =
        sum[Real](map(List.nil[T], function(a: T) { product_row_sum_fn2(right, g, f(a)) }))
    map(List.nil[T], function(a: T) { product_row_sum_fn2(right, g, f(a)) }) = List.nil[Real]
    sum[Real](List.nil[Real]) = Real.0
    product_double_sum_fn(List.nil[T], right, f, g) = Real.0
    p(List.nil[T])

    forall(head: T, tail: List[T]) {
        if p(tail) {
            map(List.cons(head, tail), f) = List.cons(f(head), map(tail, f))
            sum[Real](List.cons(f(head), map(tail, f))) = f(head) + sum[Real](map(tail, f))
            sum[Real](map(List.cons(head, tail), f)) = f(head) + sum[Real](map(tail, f))
            (f(head) + sum[Real](map(tail, f))) * sum[Real](map(right, g)) =
                f(head) * sum[Real](map(right, g)) + sum[Real](map(tail, f)) * sum[Real](map(right, g))
            sum[Real](map(List.cons(head, tail), f)) * sum[Real](map(right, g)) =
                f(head) * sum[Real](map(right, g)) + sum[Real](map(tail, f)) * sum[Real](map(right, g))
            list_sum_scalar_mul[U](f(head), right, g)
            f(head) * sum[Real](map(right, g)) = sum[Real](map(right, mul_fn(f(head), g)))
            f(head) * sum[Real](map(right, g)) = product_row_sum_fn2(right, g, f(head))
            p(tail) = (sum[Real](map(tail, f)) * sum[Real](map(right, g)) =
                product_double_sum_fn(tail, right, f, g))
            sum[Real](map(tail, f)) * sum[Real](map(right, g)) =
                product_double_sum_fn(tail, right, f, g)
            sum[Real](map(List.cons(head, tail), f)) * sum[Real](map(right, g)) =
                product_row_sum_fn2(right, g, f(head)) + product_double_sum_fn(tail, right, f, g)
            product_double_sum_fn(List.cons(head, tail), right, f, g) =
                sum[Real](map(List.cons(head, tail), function(a: T) { product_row_sum_fn2(right, g, f(a)) }))
            map_cons_general[T, Real](head, tail,
                function(a: T) { product_row_sum_fn2(right, g, f(a)) })
            map(List.cons(head, tail), function(a: T) { product_row_sum_fn2(right, g, f(a)) }) =
                List.cons(product_row_sum_fn2(right, g, f(head)),
                    map(tail, function(a: T) { product_row_sum_fn2(right, g, f(a)) }))
            sum[Real](List.cons(product_row_sum_fn2(right, g, f(head)),
                map(tail, function(a: T) { product_row_sum_fn2(right, g, f(a)) }))) =
                product_row_sum_fn2(right, g, f(head)) + sum[Real](map(tail, function(a: T) { product_row_sum_fn2(right, g, f(a)) }))
            product_double_sum_fn(List.cons(head, tail), right, f, g) =
                product_row_sum_fn2(right, g, f(head)) + sum[Real](map(tail, function(a: T) { product_row_sum_fn2(right, g, f(a)) }))
            product_double_sum_fn(tail, right, f, g) =
                sum[Real](map(tail, function(a: T) { product_row_sum_fn2(right, g, f(a)) }))
            sum[Real](map(tail, function(a: T) { product_row_sum_fn2(right, g, f(a)) })) =
                product_double_sum_fn(tail, right, f, g)
            product_double_sum_fn(List.cons(head, tail), right, f, g) =
                product_row_sum_fn2(right, g, f(head)) + product_double_sum_fn(tail, right, f, g)
            sum[Real](map(List.cons(head, tail), f)) * sum[Real](map(right, g)) =
                product_double_sum_fn(List.cons(head, tail), right, f, g)
            p(List.cons(head, tail))
        }
    }

    List.induction(function(l: List[T]) { p(l) })
    function[T0](x0: List[T0] -> Bool, x1: List[T0]) {
        not x0(List.nil[T0]) or exists(k0: T0, k1: List[T0]) {
            x0(k1) and not x0(List.cons[T0](k0, k1))
        } or x0(x1)
    }[T](p, left)
    function(x0: List[T]) {
        (sum[Real](map(x0, f)) * sum[Real](map(right, g)) =
            product_double_sum_fn(x0, right, f, g)) = p(x0)
    }(left)
    sum[Real](map(left, f)) * sum[Real](map(right, g)) = product_double_sum_fn(left, right, f, g)
}

/// Summing the mass-weighted joint value indicator over the support gives the
/// probability of the intersection of the two value events.
theorem finite_set_sum_mass_pair_value_indicator_eq_intersection_prob[T](pmf: DiscretePMF[T],
    x: T -> Real, y: T -> Real, a: Real, b: Real) {
    finite_set_sum(pmf.support, mass_pair_value_indicator(pmf.mass, x, y, a, b)) =
        pmf_event_prob(pmf, fs_intersection(rv_value_event(pmf, x, a), rv_value_event(pmf, y, b)))
} by {
    let ex = rv_value_event(pmf, x, a)
    let ey = rv_value_event(pmf, y, b)
    let inside = fs_intersection(ex, ey)
    let outside = fs_difference(pmf.support, inside)
    let mpvi = mass_pair_value_indicator(pmf.mass, x, y, a, b)

    finite_set_subset_refl[T](inside)
    inside.subset_eq(inside)
    finite_set_subset_intersection_eq[T](ex, ey, inside)
    inside.subset_eq(inside) = (inside.subset_eq(ex) and inside.subset_eq(ey))
    inside.subset_eq(ex)
    rv_value_event_subset_support(pmf, x, a)
    ex.subset_eq(pmf.support)
    finite_set_subset_trans[T](inside, ex, pmf.support)
    inside.subset_eq(pmf.support)

    finite_set_intersection_union_difference_is_self[T](pmf.support, inside)
    finite_set_intersection_is_disjoint_difference[T](pmf.support, inside)
    finite_set_intersection_eq_right_of_subset(pmf.support, inside)
    fs_intersection(pmf.support, inside) = inside
    finite_set_sum_disjoint_union[T, Real](inside, outside, mpvi)
    finite_set_sum(fs_union(inside, outside), mpvi) =
        finite_set_sum(inside, mpvi) + finite_set_sum(outside, mpvi)

    forall(t: T) {
        if inside.contains(t) {
            finite_set_intersection_contains_eq(ex, ey, t)
            inside.contains(t) = (ex.contains(t) and ey.contains(t))
            rv_value_event_contains_eq(pmf, x, a, t)
            ex.contains(t) = (pmf.support.contains(t) and x(t) = a)
            rv_value_event_contains_eq(pmf, y, b, t)
            ey.contains(t) = (pmf.support.contains(t) and y(t) = b)
            x(t) = a
            y(t) = b
            rv_pair_value_indicator_one_of_pair(x, y, a, b, t)
            rv_pair_value_indicator(x, y, a, b, t) = Real.1
            mpvi(t) = pmf.mass(t) * rv_pair_value_indicator(x, y, a, b, t)
            pmf.mass(t) * Real.1 = pmf.mass(t)
            mpvi(t) = pmf.mass(t)
        }
    }
    forall(t: T) { inside.contains(t) implies mpvi(t) = pmf.mass(t) }
    finite_set_sum_eq_of_pointwise_on_set[T, Real](inside, mpvi, pmf.mass)
    finite_set_sum(inside, mpvi) = finite_set_sum(inside, pmf.mass)

    forall(t: T) {
        if outside.contains(t) {
            finite_set_difference_contains_eq(pmf.support, inside, t)
            outside.contains(t) = (pmf.support.contains(t) and not inside.contains(t))
            pmf.support.contains(t)
            rv_value_event_contains_eq(pmf, x, a, t)
            ex.contains(t) = (pmf.support.contains(t) and x(t) = a)
            rv_value_event_contains_eq(pmf, y, b, t)
            ey.contains(t) = (pmf.support.contains(t) and y(t) = b)
            if x(t) = a {
                if y(t) = b {
                    ex.contains(t)
                    ey.contains(t)
                    inside.contains(t)
                    false
                }
                y(t) != b
                rv_pair_value_indicator_zero_of_not_pair(x, y, a, b, t)
                rv_pair_value_indicator(x, y, a, b, t) = Real.0
                mpvi(t) = pmf.mass(t) * rv_pair_value_indicator(x, y, a, b, t)
                pmf.mass(t) * Real.0 = Real.0
                mpvi(t) = Real.0
            }
            if x(t) != a {
                rv_pair_value_indicator_zero_of_not_pair(x, y, a, b, t)
                rv_pair_value_indicator(x, y, a, b, t) = Real.0
                mpvi(t) = pmf.mass(t) * rv_pair_value_indicator(x, y, a, b, t)
                pmf.mass(t) * Real.0 = Real.0
                mpvi(t) = Real.0
            }
            x(t) = a or x(t) != a
            mpvi(t) = Real.0
        }
    }
    forall(t: T) { outside.contains(t) implies mpvi(t) = Real.0 }
    finite_set_sum_zero_of_contains(outside, mpvi)
    finite_set_sum(outside, mpvi) = Real.0

    finite_set_sum(inside, pmf.mass) + Real.0 = finite_set_sum(inside, pmf.mass)
    finite_set_sum(fs_union(inside, outside), mpvi) =
        finite_set_sum(inside, pmf.mass) + Real.0
    finite_set_sum(fs_union(inside, outside), mpvi) = finite_set_sum(inside, pmf.mass)
    fs_union(fs_intersection(pmf.support, inside), outside) = pmf.support
    fs_union(inside, outside) = pmf.support
    finite_set_sum(pmf.support, mpvi) = finite_set_sum(inside, pmf.mass)
    pmf_event_prob(pmf, inside) = finite_set_sum(inside, pmf.mass)
    finite_set_sum(pmf.support, mpvi) = pmf_event_prob(pmf, inside)
    finite_set_sum(pmf.support, mpvi) = pmf_event_prob(pmf, fs_intersection(ex, ey))
    finite_set_sum(pmf.support, mpvi) =
        pmf_event_prob(pmf, fs_intersection(rv_value_event(pmf, x, a), rv_value_event(pmf, y, b)))
}

/// The row sum of joint value-weighted mass terms over the second image list.
define pair_row_sum_fn[T](mass: T -> Real, x: T -> Real, y: T -> Real, values_y: List[Real], a: Real, t: T) -> Real {
    list_sum_fn(values_y, mass_pair_expansion_term(mass, x, y, a), t)
}

/// The expectation of the product of two independent discrete random variables
/// factorizes: `E(X * Y) = E(X) * E(Y)`.
theorem discrete_expectation_product_of_independent[T](pmf: DiscretePMF[T], x: T -> Real, y: T -> Real) {
    discrete_rv_independent(pmf, x, y) implies
        discrete_expectation(pmf, pointwise_mul_fn(x, y)) =
            discrete_expectation(pmf, x) * discrete_expectation(pmf, y)
} by {
    if discrete_rv_independent(pmf, x, y) {
        let values_x = finite_fiber_partition_values(pmf.support, x)
        let values_y = finite_fiber_partition_values(pmf.support, y)
        let mpet = mass_pair_expansion_term(pmf.mass, x, y)
        let row = pair_row_sum_fn(pmf.mass, x, y, values_y)
        let mwv_xy = mass_weighted_value(pmf.mass, pointwise_mul_fn(x, y))

        forall(t: T) {
            if pmf.support.contains(t) {
                rv_expansion_at_point(pmf, x, t)
                rv_expansion_at_point(pmf, y, t)
                sum[Real](map(values_x, rv_expansion_term(x, t))) = x(t)
                sum[Real](map(values_y, rv_expansion_term(y, t))) = y(t)
                mwv_xy(t) = pmf.mass(t) * pointwise_mul_fn(x, y, t)
                pointwise_mul_fn(x, y, t) = x(t) * y(t)
                mwv_xy(t) = pmf.mass(t) * (x(t) * y(t))
                pmf.mass(t) * (x(t) * y(t)) = (pmf.mass(t) * x(t)) * y(t)
                mwv_xy(t) = (pmf.mass(t) * x(t)) * y(t)
                list_sum_scalar_mul[Real](pmf.mass(t), values_x, rv_expansion_term(x, t))
                pmf.mass(t) * sum[Real](map(values_x, rv_expansion_term(x, t))) =
                    sum[Real](map(values_x, mul_fn(pmf.mass(t), rv_expansion_term(x, t))))
                pmf.mass(t) * x(t) = sum[Real](map(values_x, mul_fn(pmf.mass(t), rv_expansion_term(x, t))))
                mwv_xy(t) =
                    sum[Real](map(values_x, mul_fn(pmf.mass(t), rv_expansion_term(x, t)))) * y(t)
                mwv_xy(t) =
                    sum[Real](map(values_x, mul_fn(pmf.mass(t), rv_expansion_term(x, t)))) * sum[Real](map(values_y, rv_expansion_term(y, t)))
                list_sum_product_of_sums[Real, Real](values_x, values_y,
                    mul_fn(pmf.mass(t), rv_expansion_term(x, t)), rv_expansion_term(y, t))
                sum[Real](map(values_x, mul_fn(pmf.mass(t), rv_expansion_term(x, t)))) * sum[Real](map(values_y, rv_expansion_term(y, t))) =
                    product_double_sum_fn(values_x, values_y,
                        mul_fn(pmf.mass(t), rv_expansion_term(x, t)), rv_expansion_term(y, t))
                mwv_xy(t) = product_double_sum_fn(values_x, values_y,
                    mul_fn(pmf.mass(t), rv_expansion_term(x, t)), rv_expansion_term(y, t))
                forall(a: Real) {
                    product_row_sum_fn2(values_y, rv_expansion_term(y, t),
                        mul_fn(pmf.mass(t), rv_expansion_term(x, t), a)) =
                        sum[Real](map(values_y, mul_fn(mul_fn(pmf.mass(t), rv_expansion_term(x, t), a),
                            rv_expansion_term(y, t))))
                    forall(b: Real) {
                        mul_fn(mul_fn(pmf.mass(t), rv_expansion_term(x, t), a), rv_expansion_term(y, t), b) =
                            mul_fn(pmf.mass(t), rv_expansion_term(x, t), a) * rv_expansion_term(y, t, b)
                        mul_fn(pmf.mass(t), rv_expansion_term(x, t), a) =
                            pmf.mass(t) * rv_expansion_term(x, t, a)
                        rv_expansion_term(x, t, a) = a * rv_value_indicator(x, t, a)
                        rv_expansion_term(y, t, b) = b * rv_value_indicator(y, t, b)
                        mul_fn(pmf.mass(t), rv_expansion_term(x, t), a) * rv_expansion_term(y, t, b) =
                            (pmf.mass(t) * (a * rv_value_indicator(x, t, a))) * (b * rv_value_indicator(y, t, b))
                        (pmf.mass(t) * (a * rv_value_indicator(x, t, a))) * (b * rv_value_indicator(y, t, b)) =
                            (pmf.mass(t) * a * rv_value_indicator(x, t, a)) * (b * rv_value_indicator(y, t, b))
                        (pmf.mass(t) * a * rv_value_indicator(x, t, a)) * (b * rv_value_indicator(y, t, b)) =
                            pmf.mass(t) * a * rv_value_indicator(x, t, a) * b * rv_value_indicator(y, t, b)
                        pmf.mass(t) * a * rv_value_indicator(x, t, a) * b * rv_value_indicator(y, t, b) =
                            (pmf.mass(t) * a * rv_value_indicator(x, t, a)) * (b * rv_value_indicator(y, t, b))
                        (pmf.mass(t) * a * rv_value_indicator(x, t, a)) * (b * rv_value_indicator(y, t, b)) =
                            (a * (pmf.mass(t) * rv_value_indicator(x, t, a))) * (b * rv_value_indicator(y, t, b))
                        (a * (pmf.mass(t) * rv_value_indicator(x, t, a))) * (b * rv_value_indicator(y, t, b)) =
                            a * ((pmf.mass(t) * rv_value_indicator(x, t, a)) * (b * rv_value_indicator(y, t, b)))
                        a * ((pmf.mass(t) * rv_value_indicator(x, t, a)) * (b * rv_value_indicator(y, t, b))) =
                            a * (b * ((pmf.mass(t) * rv_value_indicator(x, t, a)) * rv_value_indicator(y, t, b)))
                        a * (b * ((pmf.mass(t) * rv_value_indicator(x, t, a)) * rv_value_indicator(y, t, b))) =
                            a * b * ((pmf.mass(t) * rv_value_indicator(x, t, a)) * rv_value_indicator(y, t, b))
                        a * b * ((pmf.mass(t) * rv_value_indicator(x, t, a)) * rv_value_indicator(y, t, b)) =
                            a * b * (pmf.mass(t) * rv_value_indicator(x, t, a) * rv_value_indicator(y, t, b))
                        a * b * (pmf.mass(t) * rv_value_indicator(x, t, a) * rv_value_indicator(y, t, b)) =
                            a * b * (pmf.mass(t) * (rv_value_indicator(x, t, a) * rv_value_indicator(y, t, b)))
                        rv_pair_value_indicator(x, y, a, b, t) =
                            rv_value_indicator(x, t, a) * rv_value_indicator(y, t, b)
                        a * b * (pmf.mass(t) * (rv_value_indicator(x, t, a) * rv_value_indicator(y, t, b))) =
                            a * b * (pmf.mass(t) * rv_pair_value_indicator(x, y, a, b, t))
                        mass_pair_value_indicator(pmf.mass, x, y, a, b, t) =
                            pmf.mass(t) * rv_pair_value_indicator(x, y, a, b, t)
                        a * b * (pmf.mass(t) * rv_pair_value_indicator(x, y, a, b, t)) =
                            a * b * mass_pair_value_indicator(pmf.mass, x, y, a, b, t)
                        mass_pair_expansion_term(pmf.mass, x, y, a, b, t) =
                            a * b * mass_pair_value_indicator(pmf.mass, x, y, a, b, t)
                        a * b * mass_pair_value_indicator(pmf.mass, x, y, a, b, t) =
                            mass_pair_expansion_term(pmf.mass, x, y, a, b, t)
                        mul_fn(mul_fn(pmf.mass(t), rv_expansion_term(x, t), a), rv_expansion_term(y, t), b) =
                            mass_pair_expansion_term(pmf.mass, x, y, a, b, t)
                        mass_pair_expansion_term(pmf.mass, x, y, a, b, t) = mpet(a, b, t)
                        mul_fn(mul_fn(pmf.mass(t), rv_expansion_term(x, t), a), rv_expansion_term(y, t), b) =
                            mpet(a, b, t)
                    }
                    sum_map_of_pointwise[Real, Real](values_y,
                        mul_fn(mul_fn(pmf.mass(t), rv_expansion_term(x, t), a), rv_expansion_term(y, t)),
                        function(b: Real) { mpet(a, b, t) })
                    sum[Real](map(values_y, mul_fn(mul_fn(pmf.mass(t), rv_expansion_term(x, t), a),
                        rv_expansion_term(y, t)))) =
                        sum[Real](map(values_y, function(b: Real) { mpet(a, b, t) }))
                    product_row_sum_fn2(values_y, rv_expansion_term(y, t),
                        mul_fn(pmf.mass(t), rv_expansion_term(x, t), a)) =
                        sum[Real](map(values_y, function(b: Real) { mpet(a, b, t) }))
                    row(a, t) = list_sum_fn(values_y, mpet(a), t)
                    list_sum_fn(values_y, mpet(a), t) =
                        sum[Real](map(values_y, function(b: Real) { mpet(a, b, t) }))
                    sum[Real](map(values_y, function(b: Real) { mpet(a, b, t) })) =
                        list_sum_fn(values_y, mpet(a), t)
                    product_row_sum_fn2(values_y, rv_expansion_term(y, t),
                        mul_fn(pmf.mass(t), rv_expansion_term(x, t), a)) = row(a, t)
                }
                function_extensionality[Real, Real](
                    function(a: Real) { product_row_sum_fn2(values_y, rv_expansion_term(y, t),
                        mul_fn(pmf.mass(t), rv_expansion_term(x, t), a)) },
                    function(a: Real) { row(a, t) })
                function(a: Real) { product_row_sum_fn2(values_y, rv_expansion_term(y, t),
                    mul_fn(pmf.mass(t), rv_expansion_term(x, t), a)) } =
                    function(a: Real) { row(a, t) }
                sum_map_of_pointwise[Real, Real](values_x,
                    function(a: Real) { product_row_sum_fn2(values_y, rv_expansion_term(y, t),
                        mul_fn(pmf.mass(t), rv_expansion_term(x, t), a)) },
                    function(a: Real) { row(a, t) })
                sum[Real](map(values_x, function(a: Real) { product_row_sum_fn2(values_y, rv_expansion_term(y, t),
                    mul_fn(pmf.mass(t), rv_expansion_term(x, t), a)) })) =
                    sum[Real](map(values_x, function(a: Real) { row(a, t) }))
                product_double_sum_fn(values_x, values_y,
                    mul_fn(pmf.mass(t), rv_expansion_term(x, t)), rv_expansion_term(y, t)) =
                    sum[Real](map(values_x, function(a: Real) { product_row_sum_fn2(values_y, rv_expansion_term(y, t),
                        mul_fn(pmf.mass(t), rv_expansion_term(x, t), a)) }))
                product_double_sum_fn(values_x, values_y,
                    mul_fn(pmf.mass(t), rv_expansion_term(x, t)), rv_expansion_term(y, t)) =
                    sum[Real](map(values_x, function(a: Real) { row(a, t) }))
                list_sum_fn(values_x, row, t) =
                    sum[Real](map(values_x, function(a: Real) { row(a, t) }))
                sum[Real](map(values_x, function(a: Real) { row(a, t) })) =
                    list_sum_fn(values_x, row, t)
                mwv_xy(t) = list_sum_fn(values_x, row, t)
            }
        }
        forall(t: T) { pmf.support.contains(t) implies mwv_xy(t) = list_sum_fn(values_x, row, t) }
        finite_set_sum_eq_of_pointwise_on_set[T, Real](pmf.support, mwv_xy, list_sum_fn(values_x, row))
        finite_set_sum(pmf.support, mwv_xy) = finite_set_sum(pmf.support, list_sum_fn(values_x, row))
        discrete_expectation(pmf, pointwise_mul_fn(x, y)) = finite_set_sum(pmf.support, mwv_xy)
        discrete_expectation(pmf, pointwise_mul_fn(x, y)) =
            finite_set_sum(pmf.support, list_sum_fn(values_x, row))
        finite_set_sum_list_sum_swap(pmf.support, values_x, row)
        finite_set_sum(pmf.support, list_sum_fn(values_x, row)) =
            sum[Real](map(values_x, finite_set_sum_fn(pmf.support, row)))
        discrete_expectation(pmf, pointwise_mul_fn(x, y)) =
            sum[Real](map(values_x, finite_set_sum_fn(pmf.support, row)))

        forall(a: Real) {
            finite_set_sum_fn(pmf.support, row, a) = finite_set_sum(pmf.support, row(a))
            forall(t: T) {
                row(a, t) = list_sum_fn(values_y, mpet(a), t)
            }
            function_extensionality[T, Real](row(a), list_sum_fn(values_y, mpet(a)))
            row(a) = list_sum_fn(values_y, mpet(a))
            finite_set_sum(pmf.support, row(a)) = finite_set_sum(pmf.support, list_sum_fn(values_y, mpet(a)))
            finite_set_sum_list_sum_swap(pmf.support, values_y, mpet(a))
            finite_set_sum(pmf.support, list_sum_fn(values_y, mpet(a))) =
                sum[Real](map(values_y, finite_set_sum_fn(pmf.support, mpet(a))))
            finite_set_sum(pmf.support, row(a)) =
                sum[Real](map(values_y, finite_set_sum_fn(pmf.support, mpet(a))))
            finite_set_sum_fn(pmf.support, row, a) =
                sum[Real](map(values_y, finite_set_sum_fn(pmf.support, mpet(a))))

            forall(b: Real) {
                finite_set_sum_fn(pmf.support, mpet(a), b) = finite_set_sum(pmf.support, mpet(a, b))
                forall(t: T) {
                    mpet(a, b, t) = a * b * mass_pair_value_indicator(pmf.mass, x, y, a, b, t)
                    mul_fn(a * b, mass_pair_value_indicator(pmf.mass, x, y, a, b), t) =
                        (a * b) * mass_pair_value_indicator(pmf.mass, x, y, a, b, t)
                    mpet(a, b, t) = mul_fn(a * b, mass_pair_value_indicator(pmf.mass, x, y, a, b), t)
                }
                function_extensionality[T, Real](mpet(a, b),
                    mul_fn(a * b, mass_pair_value_indicator(pmf.mass, x, y, a, b)))
                mpet(a, b) = mul_fn(a * b, mass_pair_value_indicator(pmf.mass, x, y, a, b))
                finite_set_sum(pmf.support, mpet(a, b)) =
                    finite_set_sum(pmf.support, mul_fn(a * b, mass_pair_value_indicator(pmf.mass, x, y, a, b)))
                finite_set_sum_scalar_mul[T, Real](a * b, pmf.support,
                    mass_pair_value_indicator(pmf.mass, x, y, a, b))
                (a * b) * finite_set_sum(pmf.support, mass_pair_value_indicator(pmf.mass, x, y, a, b)) =
                    finite_set_sum(pmf.support, mul_fn(a * b, mass_pair_value_indicator(pmf.mass, x, y, a, b)))
                finite_set_sum(pmf.support, mpet(a, b)) =
                    (a * b) * finite_set_sum(pmf.support, mass_pair_value_indicator(pmf.mass, x, y, a, b))
                finite_set_sum_mass_pair_value_indicator_eq_intersection_prob(pmf, x, y, a, b)
                finite_set_sum(pmf.support, mass_pair_value_indicator(pmf.mass, x, y, a, b)) =
                    pmf_event_prob(pmf, fs_intersection(rv_value_event(pmf, x, a), rv_value_event(pmf, y, b)))
                (a * b) * finite_set_sum(pmf.support, mass_pair_value_indicator(pmf.mass, x, y, a, b)) =
                    (a * b) * pmf_event_prob(pmf,
                        fs_intersection(rv_value_event(pmf, x, a), rv_value_event(pmf, y, b)))
                finite_set_sum(pmf.support, mpet(a, b)) =
                    (a * b) * pmf_event_prob(pmf,
                        fs_intersection(rv_value_event(pmf, x, a), rv_value_event(pmf, y, b)))
                discrete_rv_independent(pmf, x, y) = forall(a0: Real, b0: Real) {
                    discrete_events_independent(pmf, rv_value_event(pmf, x, a0),
                        rv_value_event(pmf, y, b0))
                }
                discrete_events_independent(pmf, rv_value_event(pmf, x, a),
                    rv_value_event(pmf, y, b))
                discrete_events_independent(pmf, rv_value_event(pmf, x, a),
                    rv_value_event(pmf, y, b)) =
                    (pmf_event_prob(pmf, fs_intersection(rv_value_event(pmf, x, a),
                        rv_value_event(pmf, y, b))) =
                        pmf_event_prob(pmf, rv_value_event(pmf, x, a)) * pmf_event_prob(pmf, rv_value_event(pmf, y, b)))
                pmf_event_prob(pmf, fs_intersection(rv_value_event(pmf, x, a),
                    rv_value_event(pmf, y, b))) =
                    pmf_event_prob(pmf, rv_value_event(pmf, x, a)) * pmf_event_prob(pmf, rv_value_event(pmf, y, b))
                (a * b) * pmf_event_prob(pmf, fs_intersection(rv_value_event(pmf, x, a),
                    rv_value_event(pmf, y, b))) =
                    (a * b) * (pmf_event_prob(pmf, rv_value_event(pmf, x, a)) * pmf_event_prob(pmf, rv_value_event(pmf, y, b)))
                finite_set_sum(pmf.support, mpet(a, b)) =
                    (a * b) * (pmf_event_prob(pmf, rv_value_event(pmf, x, a)) * pmf_event_prob(pmf, rv_value_event(pmf, y, b)))
                (a * b) * (pmf_event_prob(pmf, rv_value_event(pmf, x, a)) * pmf_event_prob(pmf, rv_value_event(pmf, y, b))) =
                    a * (b * (pmf_event_prob(pmf, rv_value_event(pmf, x, a)) * pmf_event_prob(pmf, rv_value_event(pmf, y, b))))
                a * (b * (pmf_event_prob(pmf, rv_value_event(pmf, x, a)) * pmf_event_prob(pmf, rv_value_event(pmf, y, b)))) =
                    a * (pmf_event_prob(pmf, rv_value_event(pmf, x, a)) * (b * pmf_event_prob(pmf, rv_value_event(pmf, y, b))))
                a * (pmf_event_prob(pmf, rv_value_event(pmf, x, a)) * (b * pmf_event_prob(pmf, rv_value_event(pmf, y, b)))) =
                    (a * pmf_event_prob(pmf, rv_value_event(pmf, x, a))) * (b * pmf_event_prob(pmf, rv_value_event(pmf, y, b)))
                rv_value_mass_term(pmf, x, a) =
                    a * pmf_event_prob(pmf, rv_value_event(pmf, x, a))
                rv_value_mass_term(pmf, y, b) =
                    b * pmf_event_prob(pmf, rv_value_event(pmf, y, b))
                (a * pmf_event_prob(pmf, rv_value_event(pmf, x, a))) * (b * pmf_event_prob(pmf, rv_value_event(pmf, y, b))) =
                    rv_value_mass_term(pmf, x, a) * rv_value_mass_term(pmf, y, b)
                finite_set_sum(pmf.support, mpet(a, b)) =
                    rv_value_mass_term(pmf, x, a) * rv_value_mass_term(pmf, y, b)
                finite_set_sum_fn(pmf.support, mpet(a), b) =
                    rv_value_mass_term(pmf, x, a) * rv_value_mass_term(pmf, y, b)
            }
            forall(b: Real) {
                finite_set_sum_fn(pmf.support, mpet(a), b) =
                    rv_value_mass_term(pmf, x, a) * rv_value_mass_term(pmf, y, b)
            }
            sum_map_of_pointwise[Real, Real](values_y, finite_set_sum_fn(pmf.support, mpet(a)),
                function(b: Real) { rv_value_mass_term(pmf, x, a) * rv_value_mass_term(pmf, y, b) })
            sum[Real](map(values_y, finite_set_sum_fn(pmf.support, mpet(a)))) =
                sum[Real](map(values_y,
                    function(b: Real) { rv_value_mass_term(pmf, x, a) * rv_value_mass_term(pmf, y, b) }))
            product_row_sum_fn2(values_y, rv_value_mass_term(pmf, y), rv_value_mass_term(pmf, x, a)) =
                sum[Real](map(values_y, mul_fn(rv_value_mass_term(pmf, x, a), rv_value_mass_term(pmf, y))))
            forall(b: Real) {
                mul_fn(rv_value_mass_term(pmf, x, a), rv_value_mass_term(pmf, y), b) =
                    rv_value_mass_term(pmf, x, a) * rv_value_mass_term(pmf, y, b)
            }
            sum_map_of_pointwise[Real, Real](values_y,
                mul_fn(rv_value_mass_term(pmf, x, a), rv_value_mass_term(pmf, y)),
                function(b: Real) { rv_value_mass_term(pmf, x, a) * rv_value_mass_term(pmf, y, b) })
            sum[Real](map(values_y, mul_fn(rv_value_mass_term(pmf, x, a), rv_value_mass_term(pmf, y)))) =
                sum[Real](map(values_y,
                    function(b: Real) { rv_value_mass_term(pmf, x, a) * rv_value_mass_term(pmf, y, b) }))
            sum[Real](map(values_y, finite_set_sum_fn(pmf.support, mpet(a)))) =
                product_row_sum_fn2(values_y, rv_value_mass_term(pmf, y), rv_value_mass_term(pmf, x, a))
            finite_set_sum_fn(pmf.support, row, a) =
                product_row_sum_fn2(values_y, rv_value_mass_term(pmf, y), rv_value_mass_term(pmf, x, a))
        }
        forall(a: Real) {
            finite_set_sum_fn(pmf.support, row, a) =
                product_row_sum_fn2(values_y, rv_value_mass_term(pmf, y), rv_value_mass_term(pmf, x, a))
        }
        sum_map_of_pointwise[Real, Real](values_x, finite_set_sum_fn(pmf.support, row),
            function(a: Real) { product_row_sum_fn2(values_y, rv_value_mass_term(pmf, y),
                rv_value_mass_term(pmf, x, a)) })
        sum[Real](map(values_x, finite_set_sum_fn(pmf.support, row))) =
            sum[Real](map(values_x, function(a: Real) { product_row_sum_fn2(values_y, rv_value_mass_term(pmf, y),
                rv_value_mass_term(pmf, x, a)) }))
        product_double_sum_fn(values_x, values_y, rv_value_mass_term(pmf, x), rv_value_mass_term(pmf, y)) =
            sum[Real](map(values_x, function(a: Real) { product_row_sum_fn2(values_y, rv_value_mass_term(pmf, y),
                rv_value_mass_term(pmf, x, a)) }))
        sum[Real](map(values_x, function(a: Real) { product_row_sum_fn2(values_y, rv_value_mass_term(pmf, y),
            rv_value_mass_term(pmf, x, a)) })) =
            product_double_sum_fn(values_x, values_y, rv_value_mass_term(pmf, x), rv_value_mass_term(pmf, y))
        sum[Real](map(values_x, finite_set_sum_fn(pmf.support, row))) =
            product_double_sum_fn(values_x, values_y, rv_value_mass_term(pmf, x), rv_value_mass_term(pmf, y))
        discrete_expectation(pmf, pointwise_mul_fn(x, y)) =
            product_double_sum_fn(values_x, values_y, rv_value_mass_term(pmf, x), rv_value_mass_term(pmf, y))
        list_sum_product_of_sums[Real, Real](values_x, values_y, rv_value_mass_term(pmf, x),
            rv_value_mass_term(pmf, y))
        sum[Real](map(values_x, rv_value_mass_term(pmf, x))) * sum[Real](map(values_y, rv_value_mass_term(pmf, y))) =
            product_double_sum_fn(values_x, values_y, rv_value_mass_term(pmf, x), rv_value_mass_term(pmf, y))
        product_double_sum_fn(values_x, values_y, rv_value_mass_term(pmf, x), rv_value_mass_term(pmf, y)) =
            sum[Real](map(values_x, rv_value_mass_term(pmf, x))) * sum[Real](map(values_y, rv_value_mass_term(pmf, y)))
        discrete_expectation(pmf, pointwise_mul_fn(x, y)) =
            sum[Real](map(values_x, rv_value_mass_term(pmf, x))) * sum[Real](map(values_y, rv_value_mass_term(pmf, y)))
        discrete_expectation_eq_value_sum(pmf, x)
        discrete_expectation(pmf, x) = sum[Real](map(values_x, rv_value_mass_term(pmf, x)))
        discrete_expectation_eq_value_sum(pmf, y)
        discrete_expectation(pmf, y) = sum[Real](map(values_y, rv_value_mass_term(pmf, y)))
        discrete_expectation(pmf, pointwise_mul_fn(x, y)) =
            discrete_expectation(pmf, x) * discrete_expectation(pmf, y)
    }
}

/// Expectation is monotone under pointwise order everywhere (a global corollary
/// of the support-restricted monotonicity in `discrete_pmf`).
theorem discrete_expectation_monotone_global[T](pmf: DiscretePMF[T], x: T -> Real, y: T -> Real) {
    (forall(t: T) { x(t) <= y(t) }) implies discrete_expectation(pmf, x) <= discrete_expectation(pmf, y)
} by {
    if forall(t: T) { x(t) <= y(t) } {
        forall(t: T) {
            if pmf.support.contains(t) {
                x(t) <= y(t)
            }
        }
        forall(t: T) { pmf.support.contains(t) implies x(t) <= y(t) }
        discrete_expectation_monotone(pmf, x, y)
        discrete_expectation(pmf, x) <= discrete_expectation(pmf, y)
    }
}

/// Expectation is extensional for random variables that agree pointwise on the support.
theorem discrete_expectation_eq_of_pointwise_eq_on_support[T](pmf: DiscretePMF[T],
    x: T -> Real, y: T -> Real) {
    (forall(t: T) { pmf.support.contains(t) implies x(t) = y(t) }) implies
        discrete_expectation(pmf, x) = discrete_expectation(pmf, y)
} by {
    if forall(t: T) { pmf.support.contains(t) implies x(t) = y(t) } {
        forall(t: T) {
            if pmf.support.contains(t) {
                mass_weighted_value(pmf.mass, x, t) = pmf.mass(t) * x(t)
                x(t) = y(t)
                pmf.mass(t) * y(t) = mass_weighted_value(pmf.mass, y, t)
                mass_weighted_value(pmf.mass, x, t) = mass_weighted_value(pmf.mass, y, t)
            }
        }
        forall(t: T) {
            pmf.support.contains(t) implies mass_weighted_value(pmf.mass, x, t) = mass_weighted_value(pmf.mass, y, t)
        }
        finite_set_sum_eq_of_pointwise_on_set[T, Real](pmf.support,
            mass_weighted_value(pmf.mass, x), mass_weighted_value(pmf.mass, y))
        finite_set_sum(pmf.support, mass_weighted_value(pmf.mass, x)) =
            finite_set_sum(pmf.support, mass_weighted_value(pmf.mass, y))
        discrete_expectation(pmf, x) = finite_set_sum(pmf.support, mass_weighted_value(pmf.mass, x))
        discrete_expectation(pmf, y) = finite_set_sum(pmf.support, mass_weighted_value(pmf.mass, y))
        discrete_expectation(pmf, x) = discrete_expectation(pmf, y)
    }
}

// The Bernoulli pmf bundled from the canonical data satisfies the Bernoulli predicate.
// COMMENTED OUT: extracting the mass field from the bundled constructor equality
// (`DiscretePMF.new(support, mass) = Option.some(pmf)` implies `pmf.mass = mass`)
// is not automatic for function-valued fields in the current prover, and the
// statement times out. The `is_bernoulli_pmf` predicate used by the theorems above
// is instead meant to be established directly from the canonical data
// (`pmf.support = bernoulli_support`, `pmf.mass = bernoulli_mass(p)`).
// theorem bernoulli_pmf_exists_is_bernoulli(p: Real, pmf: DiscretePMF[Bool]) {
//     DiscretePMF[Bool].new(bernoulli_support, bernoulli_mass(p)) = Option.some(pmf) implies
//         is_bernoulli_pmf(p, pmf)
// } by {
//     if DiscretePMF[Bool].new(bernoulli_support, bernoulli_mass(p)) = Option.some(pmf) {
//         bernoulli_pmf_new_support(p, pmf)
//         pmf.support = bernoulli_support
//         forall(b: Bool) {
//             if b {
//                 pmf.mass(b) = bernoulli_mass(p, b)
//             }
//             if not b {
//                 pmf.mass(b) = bernoulli_mass(p, b)
//             }
//             b or not b
//             pmf.mass(b) = bernoulli_mass(p, b)
//         }
//         is_bernoulli_pmf(p, pmf) =
//             (pmf.support = bernoulli_support and forall(b: Bool) { pmf.mass(b) = bernoulli_mass(p, b) })
//         is_bernoulli_pmf(p, pmf)
//     }
// }
