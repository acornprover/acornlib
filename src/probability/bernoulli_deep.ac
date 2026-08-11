from real import Real
from finite_set import FiniteSet, fs_intersection, fs_difference
from probability.discrete_pmf import DiscretePMF, discrete_expectation, discrete_variance,
    square_fn, finite_event_indicator, finite_event_indicator_one_of_contains,
    finite_event_indicator_zero_of_not_contains, pmf_event_prob,
    pmf_event_prob_eq_support_intersection, pmf_event_prob_complement
from probability.discrete_probability_ext import is_bernoulli_pmf, bool_true_event,
    bernoulli_indicator_expectation, bernoulli_indicator_expectation_square,
    bernoulli_indicator_variance, rv_value_event, rv_value_event_indicator_one,
    rv_value_event_indicator_zero

/// The deep facts about the Bernoulli distribution: the expectation `E(X) = p`,
/// the variance `Var(X) = p(1 - p)`, the second moment `E(X^2) = p`, and the
/// identification of the indicator of an event as a Bernoulli random variable
/// with parameter `P(A)`.
///
/// Here the Bernoulli random variable is the indicator of the Boolean event
/// `{true}` under a pmf satisfying `is_bernoulli_pmf(p, pmf)`: mass `p` at
/// `true` and mass `1 - p` at `false`.  The three moment theorems restate the
/// computations of `discrete_probability_ext`; the indicator theorems are the
/// new content of this file.

/// The expectation of the Bernoulli random variable `X = 1_{true}` under a
/// Bernoulli(p) pmf is `p`.
theorem bernoulli_deep_expectation(p: Real, pmf: DiscretePMF[Bool]) {
    is_bernoulli_pmf(p, pmf) implies
        discrete_expectation(pmf, finite_event_indicator(bool_true_event)) = p
} by {
    bernoulli_indicator_expectation(p, pmf)
}

/// The variance of the Bernoulli random variable `X = 1_{true}` under a
/// Bernoulli(p) pmf is `p * (1 - p)`.
theorem bernoulli_deep_variance(p: Real, pmf: DiscretePMF[Bool]) {
    is_bernoulli_pmf(p, pmf) implies
        discrete_variance(pmf, finite_event_indicator(bool_true_event)) = p * (Real.1 - p)
} by {
    bernoulli_indicator_variance(p, pmf)
}

/// The second moment of the Bernoulli random variable `X = 1_{true}` under a
/// Bernoulli(p) pmf is `p`: since `X` is 0/1-valued, `X^2 = X` pointwise and
/// hence `E(X^2) = E(X) = p`.
theorem bernoulli_deep_second_moment(p: Real, pmf: DiscretePMF[Bool]) {
    is_bernoulli_pmf(p, pmf) implies
        discrete_expectation(pmf, square_fn(finite_event_indicator(bool_true_event))) = p
} by {
    bernoulli_indicator_expectation_square(p, pmf)
}

/// A finite-event indicator is 0/1-valued: at every point of the underlying type
/// it takes the value `0` or the value `1`.
theorem bernoulli_deep_indicator_values[T](a: FiniteSet[T], x: T) {
    finite_event_indicator(a, x) = Real.0 or finite_event_indicator(a, x) = Real.1
} by {
    if a.contains(x) {
        finite_event_indicator_one_of_contains(a, x)
        finite_event_indicator(a, x) = Real.1
        finite_event_indicator(a, x) = Real.0 or finite_event_indicator(a, x) = Real.1
    }
    if not a.contains(x) {
        finite_event_indicator_zero_of_not_contains(a, x)
        finite_event_indicator(a, x) = Real.0
        finite_event_indicator(a, x) = Real.0 or finite_event_indicator(a, x) = Real.1
    }
    a.contains(x) or not a.contains(x)
}

/// The indicator of a finite event takes the value `1` exactly on the event, so
/// the probability that `1_A = 1` is the probability of `A` itself.  This is the
/// identifying property of a Bernoulli(P(A)) random variable.
theorem bernoulli_deep_indicator_one_prob[T](pmf: DiscretePMF[T], a: FiniteSet[T]) {
    pmf_event_prob(pmf, rv_value_event(pmf, finite_event_indicator(a), Real.1)) =
        pmf_event_prob(pmf, a)
} by {
    rv_value_event_indicator_one(pmf, a)
    rv_value_event(pmf, finite_event_indicator(a), Real.1) = fs_intersection(pmf.support, a)
    pmf_event_prob_eq_support_intersection(pmf, a)
    pmf_event_prob(pmf, a) = pmf_event_prob(pmf, fs_intersection(pmf.support, a))
    pmf_event_prob(pmf, fs_intersection(pmf.support, a)) = pmf_event_prob(pmf, a)
    pmf_event_prob(pmf, rv_value_event(pmf, finite_event_indicator(a), Real.1)) =
        pmf_event_prob(pmf, a)
}

/// The indicator of a finite event takes the value `0` off the event, so the
/// probability that `1_A = 0` is `1 - P(A)`, the complementary parameter of the
/// Bernoulli(P(A)) law.
theorem bernoulli_deep_indicator_zero_prob[T](pmf: DiscretePMF[T], a: FiniteSet[T]) {
    pmf_event_prob(pmf, rv_value_event(pmf, finite_event_indicator(a), Real.0)) =
        Real.1 - pmf_event_prob(pmf, a)
} by {
    rv_value_event_indicator_zero(pmf, a)
    rv_value_event(pmf, finite_event_indicator(a), Real.0) = fs_difference(pmf.support, a)
    pmf_event_prob_complement(pmf, a)
    pmf_event_prob(pmf, fs_difference(pmf.support, a)) =
        Real.1 - pmf_event_prob(pmf, fs_intersection(pmf.support, a))
    pmf_event_prob_eq_support_intersection(pmf, a)
    pmf_event_prob(pmf, a) = pmf_event_prob(pmf, fs_intersection(pmf.support, a))
    pmf_event_prob(pmf, fs_intersection(pmf.support, a)) = pmf_event_prob(pmf, a)
    Real.1 - pmf_event_prob(pmf, fs_intersection(pmf.support, a)) =
        Real.1 - pmf_event_prob(pmf, a)
    pmf_event_prob(pmf, fs_difference(pmf.support, a)) = Real.1 - pmf_event_prob(pmf, a)
}

// The sum of `n` independent Bernoulli(p) random variables is Binomial(n, p):
// the mass of the sum at `k` is C(n, k) * p^k * (1 - p)^(n - k).  This is the
// classical next step after the single-variable facts above.
//
// COMMENTED OUT: the library has no Binomial distribution over sequences of
// Boolean-valued random variables (the `binomial` module is about binomial
// coefficients in commutative rings, not the probability law), and no product
// construction that would form the joint pmf of `n` independent draws, so the
// statement cannot be expressed yet.  Once a Binomial pmf and a product-of-pmfs
// construction exist, the theorem should read:
//
// theorem bernoulli_deep_sum_is_binomial(n: Nat, p: Real) {
//     Real.0 <= p and p <= Real.1 implies
//         sum_of_n_independent_bernoulli_is_binomial(n, p)
// } by {
//     // Induction on n: one Bernoulli draw contributes E(X) = p and
//     // Var(X) = p(1 - p) (bernoulli_deep_expectation, bernoulli_deep_variance);
//     // the Binomial mass follows from the convolution of the Bernoulli masses.
// }
