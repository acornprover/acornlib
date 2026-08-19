/// Moment generating functions and higher moments for discrete probability
/// mass functions.
///
/// The library's `DiscretePMF` bundles a mass function with a finite support
/// (see `discrete_pmf.ac`), so the moment generating function below is defined
/// for finite-support laws; the exponential series machinery needed for
/// infinite-support laws such as the geometric or Poisson distributions is
/// developed in `geometric.ac` and `poisson.ac`.

from nat import Nat
from real import Real, exp_zero
from finite_set import FiniteSet, finite_set_sum, finite_set_sum_scalar_mul, finite_set_has_unique_list, fs_from_list, finite_set_sum_eq_list_sum
from list import List, map, sum, sum_map_of_pointwise
from data.basic.set import list_set, list_set_contains_eq
from algebra.semigroup import mul_fn
from data.basic.functions import function_extensionality
from probability.discrete_pmf import DiscretePMF, discrete_expectation, square_fn, discrete_variance, discrete_variance_formula, const_real_fn, discrete_pmf_total_mass

/// The pointwise exponential `x ↦ (t x(x)).exp` of a real-valued random
/// variable, weighted by the mass function.
define mgf_weighted_value[T](pmf: DiscretePMF[T], t: Real, x: T -> Real, s: T) -> Real {
    pmf.mass(s) * (t * x(s)).exp
}

/// The moment generating function of a discrete random variable at `t`:
/// M(t) = E[e^{t X}] = Σ_x p(x) e^{t x(x)}.
define discrete_mgf[T](pmf: DiscretePMF[T], t: Real, x: T -> Real) -> Real {
    finite_set_sum(pmf.support, function(v: T) { mgf_weighted_value(pmf, t, x, v) })
}

/// The moment generating function at zero is one: M(0) = E[1] = 1, because
/// e^0 = 1 on every outcome.
theorem discrete_mgf_zero_eq_one[T](pmf: DiscretePMF[T], x: T -> Real) {
    discrete_mgf(pmf, Real.0, x) = Real.1
} by {
    forall(v: T) {
        mgf_weighted_value(pmf, Real.0, x, v) = pmf.mass(v) * (Real.0 * x(v)).exp
        Real.0 * x(v) = Real.0
        (Real.0 * x(v)).exp = (Real.0).exp
        exp_zero
        Real.1 = (Real.0).exp
        (Real.0 * x(v)).exp = Real.1
        mgf_weighted_value(pmf, Real.0, x, v) = pmf.mass(v) * Real.1
        pmf.mass(v) * Real.1 = pmf.mass(v)
        mgf_weighted_value(pmf, Real.0, x, v) = pmf.mass(v)
    }
    finite_set_has_unique_list(pmf.support)
    let items: List[T] satisfy {
        fs_from_list(items) = pmf.support and items.is_unique
    }
    forall(v: T) {
        if items.contains(v) {
            fs_from_list(items).underlying_set = list_set(items)
            list_set_contains_eq(items, v)
            pmf.support.contains(v)
            mgf_weighted_value(pmf, Real.0, x, v) = pmf.mass(v)
        }
    }
    forall(v: T) { items.contains(v) implies mgf_weighted_value(pmf, Real.0, x, v) = pmf.mass(v) }
    sum_map_of_pointwise(items, function(v: T) { mgf_weighted_value(pmf, Real.0, x, v) }, pmf.mass)
    sum[Real](map(items, function(v: T) { mgf_weighted_value(pmf, Real.0, x, v) })) =
        sum[Real](map(items, pmf.mass))
    finite_set_sum_eq_list_sum[T, Real](items, function(v: T) { mgf_weighted_value(pmf, Real.0, x, v) })
    finite_set_sum(fs_from_list(items), function(v: T) { mgf_weighted_value(pmf, Real.0, x, v) }) =
        sum[Real](map(items, function(v: T) { mgf_weighted_value(pmf, Real.0, x, v) }))
    finite_set_sum_eq_list_sum[T, Real](items, pmf.mass)
    finite_set_sum(fs_from_list(items), pmf.mass) = sum[Real](map(items, pmf.mass))
    finite_set_sum(pmf.support, function(v: T) { mgf_weighted_value(pmf, Real.0, x, v) }) =
        finite_set_sum(pmf.support, pmf.mass)
    discrete_pmf_total_mass(pmf)
    finite_set_sum(pmf.support, pmf.mass) = Real.1
    discrete_mgf(pmf, Real.0, x) =
        finite_set_sum(pmf.support, function(v: T) { mgf_weighted_value(pmf, Real.0, x, v) })
    discrete_mgf(pmf, Real.0, x) = Real.1
}

/// The second moment of a discrete random variable: E[X^2].
define discrete_second_moment[T](pmf: DiscretePMF[T], x: T -> Real) -> Real {
    discrete_expectation(pmf, square_fn(x))
}

/// The second moment decomposes into variance plus the square of the mean:
/// E[X^2] = Var(X) + E[X]^2.
theorem discrete_second_moment_eq_variance_add_square[T](pmf: DiscretePMF[T], x: T -> Real) {
    discrete_second_moment(pmf, x) = discrete_variance(pmf, x) + discrete_expectation(pmf, x) * discrete_expectation(pmf, x)
} by {
    discrete_variance_formula(pmf, x)
    discrete_variance(pmf, x) = discrete_expectation(pmf, square_fn(x)) - discrete_expectation(pmf, x) * discrete_expectation(pmf, x)
    (discrete_expectation(pmf, square_fn(x)) - discrete_expectation(pmf, x) * discrete_expectation(pmf, x)) + discrete_expectation(pmf, x) * discrete_expectation(pmf, x) =
        discrete_expectation(pmf, square_fn(x))
    discrete_variance(pmf, x) + discrete_expectation(pmf, x) * discrete_expectation(pmf, x) =
        discrete_expectation(pmf, square_fn(x))
    discrete_second_moment(pmf, x) = discrete_expectation(pmf, square_fn(x))
    discrete_second_moment(pmf, x) = discrete_variance(pmf, x) + discrete_expectation(pmf, x) * discrete_expectation(pmf, x)
}

/// The moment generating function of a constant random variable at any `t` is
/// e^{t c}: M(t) = E[e^{t c}] = e^{t c}.
theorem discrete_mgf_const[T](pmf: DiscretePMF[T], t: Real, c: Real) {
    discrete_mgf(pmf, t, const_real_fn[T](c)) = (t * c).exp
} by {
    forall(v: T) {
        mgf_weighted_value(pmf, t, const_real_fn[T](c), v) = pmf.mass(v) * (t * const_real_fn[T](c, v)).exp
        const_real_fn[T](c, v) = c
        (t * const_real_fn[T](c, v)).exp = (t * c).exp
        mgf_weighted_value(pmf, t, const_real_fn[T](c), v) = pmf.mass(v) * (t * c).exp
        mul_fn((t * c).exp, pmf.mass, v) = (t * c).exp * pmf.mass(v)
        pmf.mass(v) * (t * c).exp = mul_fn((t * c).exp, pmf.mass, v)
        mgf_weighted_value(pmf, t, const_real_fn[T](c), v) = mul_fn((t * c).exp, pmf.mass, v)
    }
    function_extensionality[T, Real](function(v: T) { mgf_weighted_value(pmf, t, const_real_fn[T](c), v) },
        mul_fn((t * c).exp, pmf.mass))
    finite_set_sum_scalar_mul[T, Real]((t * c).exp, pmf.support, pmf.mass)
    (t * c).exp * finite_set_sum(pmf.support, pmf.mass) =
        finite_set_sum(pmf.support, mul_fn((t * c).exp, pmf.mass))
    discrete_pmf_total_mass(pmf)
    finite_set_sum(pmf.support, pmf.mass) = Real.1
    (t * c).exp * Real.1 = (t * c).exp
    discrete_mgf(pmf, t, const_real_fn[T](c)) =
        finite_set_sum(pmf.support, function(v: T) { mgf_weighted_value(pmf, t, const_real_fn[T](c), v) })
    discrete_mgf(pmf, t, const_real_fn[T](c)) = (t * c).exp
}

// The moment generating function of the geometric distribution (see
// geometric.ac) is M(t) = p / (1 - (1-p) e^t) for t < -ln(1-p).  Formally the
// mgf series is Σ_j p (1-p)^j e^{t j} = p / (1 - (1-p) e^t), a geometric
// series in the ratio (1-p) e^t; the verification of this exponential series
// identity is left as a statement here, in the style of the general binomial
// moments in binomial.ac.  Differentiating at t = 0 recovers E[X] = (1-p)/p
// and Var(X) = (1-p)/p^2.
