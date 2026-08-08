from finite_set import FiniteSet, fs_empty, fs_from_list, fs_union, fs_intersection,
    fs_difference, finite_set_ext, finite_set_eq_of_underlying_set_eq,
    finite_set_intersection_comm, finite_set_intersection_contains_eq,
    finite_set_intersection_empty_right, finite_set_subset_refl,
    finite_set_subset_intersection_eq,
    finite_set_intersection_union_difference_is_self,
    finite_set_intersection_is_disjoint_difference, finite_set_difference_contains_eq,
    finite_set_has_unique_list
from finite_set import finite_set_sum, finite_set_sum_empty, finite_set_sum_disjoint_union,
    finite_set_sum_add, finite_set_sum_scalar_mul, finite_set_sum_eq_list_sum
from data.finite.finite_set_sum_real import pointwise_le, finite_set_sum_le, finite_set_sum_nonneg
from list import List, map, sum
from real import Real, square_nonneg
from data.basic.set import Set, set_ext, list_set, list_set_contains_eq, union_with_difference_decomp_rev,
    intersection_comm, intersection_with_superset_is_self, intersection_contains_right,
    difference_contains_not_right
from algebra.add_semigroup import add_fn
from algebra.semigroup import mul_fn
from algebra.ring.ring import mul_neg_neg
from data.basic.functions import function_extensionality
from algebra.add_ordered_group import add_le_add_left
from order import lt_imp_lte
from ordered_field import zero_is_smaller_than_one

/// Lists mapped by functions that agree on all contained elements are equal.
lemma list_map_eq_of_contains[T, U](items: List[T], f: T -> U, g: T -> U) {
    (forall(x: T) { items.contains(x) implies f(x) = g(x) }) implies map(items, f) = map(items, g)
} by {
    define p(xs: List[T]) -> Bool {
        (forall(x: T) { xs.contains(x) implies f(x) = g(x) }) implies map(xs, f) = map(xs, g)
    }

    p(List.nil[T])

    forall(head: T, tail: List[T]) {
        if p(tail) {
            if forall(x: T) { List.cons(head, tail).contains(x) implies f(x) = g(x) } {
                List.cons(head, tail).contains(head)
                f(head) = g(head)
                forall(x: T) {
                    if tail.contains(x) {
                        List.cons(head, tail).contains(x)
                        f(x) = g(x)
                    }
                }
                forall(x: T) { tail.contains(x) implies f(x) = g(x) }
                function(x0: List[T]) { not forall(x1: T) { not x1 ∈ x0 or f(x1) = g(x1) } or not p(x0) or map[T, U](x0, g) = map[T, U](x0, f) }(tail)
                map(tail, f) = map(tail, g)
                map(List.cons(head, tail), f) = map(List.cons(head, tail), g)
            }
            p(List.cons(head, tail))
        }
    }

    List.induction(function(xs: List[T]) {
        p(xs)
    })
    function[T0](x0: List[T0] -> Bool, x1: List[T0]) {
        not x0(List.nil[T0]) or exists(k0: T0, k1: List[T0]) {
            x0(k1) and not x0(List.cons[T0](k0, k1))
        } or x0(x1)
    }[T](p, items)
    function(x0: List[T]) { (exists(k0: T) { k0 ∈ x0 and f(k0) != g(k0) } or map[T, U](x0, f) = map[T, U](x0, g)) = p(x0) }(items)
    function(x0: List[T]) { not forall(x1: T) { not x1 ∈ x0 or f(x1) = g(x1) } or map[T, U](x0, g) = map[T, U](x0, f) }(items)
}

/// True if `mass` is nonnegative on every point of `T`, vanishes outside `support`,
/// and sums to one over `support`.
define is_pmf[T](support: FiniteSet[T], mass: T -> Real) -> Bool {
    (forall(x: T) { Real.0 <= mass(x) })
        and (forall(x: T) { not support.contains(x) implies mass(x) = Real.0 })
        and finite_set_sum(support, mass) = Real.1
}

/// A discrete probability mass function on a finite support: a finite set of outcomes
/// together with a nonnegative mass function whose total over the support is one.
structure DiscretePMF[T] {
    /// The finite set of outcomes that the mass function ranges over.
    support: FiniteSet[T]
    /// The probability mass assigned to each element of `T`.
    mass: T -> Real
} constraint {
    is_pmf(support, mass)
}

/// The mass function of a discrete PMF is nonnegative everywhere.
theorem discrete_pmf_mass_nonneg[T](pmf: DiscretePMF[T], x: T) {
    Real.0 <= pmf.mass(x)
} by {
    is_pmf(pmf.support, pmf.mass) =
        ((forall(z: T) { Real.0 <= pmf.mass(z) })
            and (forall(z: T) { not pmf.support.contains(z) implies pmf.mass(z) = Real.0 })
            and finite_set_sum(pmf.support, pmf.mass) = Real.1)
    forall(z: T) { Real.0 <= pmf.mass(z) }
}

/// The mass of a discrete PMF vanishes outside its support.
theorem discrete_pmf_mass_zero_outside[T](pmf: DiscretePMF[T], x: T) {
    not pmf.support.contains(x) implies pmf.mass(x) = Real.0
} by {
    is_pmf(pmf.support, pmf.mass) =
        ((forall(z: T) { Real.0 <= pmf.mass(z) })
            and (forall(z: T) { not pmf.support.contains(z) implies pmf.mass(z) = Real.0 })
            and finite_set_sum(pmf.support, pmf.mass) = Real.1)
    forall(z: T) { not pmf.support.contains(z) implies pmf.mass(z) = Real.0 }
}

/// The total mass of a discrete PMF over its support equals one.
theorem discrete_pmf_total_mass[T](pmf: DiscretePMF[T]) {
    finite_set_sum(pmf.support, pmf.mass) = Real.1
} by {
}

/// Discrete PMF extensionality: equal supports and equal mass functions imply equal PMFs.
theorem discrete_pmf_ext[T](a: DiscretePMF[T], b: DiscretePMF[T]) {
    a.support = b.support and a.mass = b.mass implies a = b
}

/// The probability that the outcome lies in `event` is the sum of `mass` over `event`.
define pmf_event_prob[T](pmf: DiscretePMF[T], event: FiniteSet[T]) -> Real {
    finite_set_sum(event, pmf.mass)
}

/// The Real-valued indicator of a finite event.
define finite_event_indicator[T](event: FiniteSet[T], x: T) -> Real {
    if event.contains(x) { Real.1 } else { Real.0 }
}

/// The indicator is one on the event.
theorem finite_event_indicator_one_of_contains[T](event: FiniteSet[T], x: T) {
    event.contains(x) implies finite_event_indicator(event, x) = Real.1
} by {
    if event.contains(x) {
        finite_event_indicator(event, x) = Real.1
    }
}

/// The indicator is zero off the event.
theorem finite_event_indicator_zero_of_not_contains[T](event: FiniteSet[T], x: T) {
    not event.contains(x) implies finite_event_indicator(event, x) = Real.0
} by {
    if not event.contains(x) {
        finite_event_indicator(event, x) = Real.0
    }
}

lemma real_zero_le_one {
    Real.0 <= Real.1
} by {
    zero_is_smaller_than_one[Real]
    lt_imp_lte[Real](Real.0, Real.1)
}

/// A finite-event indicator is nonnegative.
theorem finite_event_indicator_nonneg[T](event: FiniteSet[T], x: T) {
    Real.0 <= finite_event_indicator(event, x)
} by {
    if event.contains(x) {
        finite_event_indicator_one_of_contains(event, x)
        real_zero_le_one
        Real.0 <= finite_event_indicator(event, x)
    }
    if not event.contains(x) {
        finite_event_indicator_zero_of_not_contains(event, x)
        Real.0 <= finite_event_indicator(event, x)
    }
    event.contains(x) or not event.contains(x)
}

/// A finite-event indicator is bounded above by one.
theorem finite_event_indicator_le_one[T](event: FiniteSet[T], x: T) {
    finite_event_indicator(event, x) <= Real.1
} by {
    if event.contains(x) {
        finite_event_indicator_one_of_contains(event, x)
        finite_event_indicator(event, x) = Real.1
        finite_event_indicator(event, x) <= Real.1
    }
    if not event.contains(x) {
        finite_event_indicator_zero_of_not_contains(event, x)
        real_zero_le_one
        finite_event_indicator(event, x) <= Real.1
    }
    event.contains(x) or not event.contains(x)
}

/// The pointwise product of a mass function and a real-valued random variable.
define mass_weighted_value[T](mass: T -> Real, rv: T -> Real, x: T) -> Real {
    mass(x) * rv(x)
}

/// The expectation of a real-valued random variable `rv` under a discrete pmf:
/// the sum over the support of `mass(x) * rv(x)`.
define discrete_expectation[T](pmf: DiscretePMF[T], rv: T -> Real) -> Real {
    finite_set_sum(pmf.support, mass_weighted_value(pmf.mass, rv))
}

/// Expectation is additive in the random variable:
/// `E(X + Y) = E(X) + E(Y)` for discrete random variables `X` and `Y`.
theorem discrete_expectation_add[T](pmf: DiscretePMF[T], x: T -> Real, y: T -> Real) {
    discrete_expectation(pmf, add_fn(x, y))
        = discrete_expectation(pmf, x) + discrete_expectation(pmf, y)
} by {
    let f = mass_weighted_value(pmf.mass, x)
    let g = mass_weighted_value(pmf.mass, y)
    let h = mass_weighted_value(pmf.mass, add_fn(x, y))
    forall(t: T) {
        h(t) = pmf.mass(t) * add_fn(x, y, t)
        add_fn(x, y, t) = x(t) + y(t)
        h(t) = pmf.mass(t) * x(t) + pmf.mass(t) * y(t)
        f(t) = pmf.mass(t) * x(t)
        g(t) = pmf.mass(t) * y(t)
        add_fn(f, g, t) = f(t) + g(t)
        h(t) = add_fn(f, g, t)
    }
    function_extensionality[T, Real](h, add_fn(f, g))
    finite_set_sum_add[T, Real](pmf.support, f, g)
}

/// Expectation is linear in scalar multiplication of the random variable:
/// `E(c * X) = c * E(X)` for any real constant `c`.
theorem discrete_expectation_scalar_mul[T](pmf: DiscretePMF[T], c: Real, x: T -> Real) {
    discrete_expectation(pmf, mul_fn(c, x)) = c * discrete_expectation(pmf, x)
} by {
    let f = mass_weighted_value(pmf.mass, x)
    let h = mass_weighted_value(pmf.mass, mul_fn(c, x))
    forall(t: T) {
        mul_fn(c, x, t) = c * x(t)
        pmf.mass(t) * (c * x(t)) = c * (pmf.mass(t) * x(t))
        h(t) = c * f(t)
        h(t) = mul_fn(c, f, t)
    }
    function_extensionality[T, Real](h, mul_fn(c, f))
    finite_set_sum_scalar_mul[T, Real](c, pmf.support, f)
}

/// The pointwise square of a real-valued function.
define square_fn[T](x: T -> Real, t: T) -> Real {
    x(t) * x(t)
}

/// The pointwise squared deviation of `x` from `mu`.
define square_dev[T](x: T -> Real, mu: Real, t: T) -> Real {
    (x(t) - mu) * (x(t) - mu)
}

/// A squared deviation is nonnegative.
theorem square_dev_nonneg[T](x: T -> Real, mu: Real, t: T) {
    Real.0 <= square_dev(x, mu, t)
} by {
    square_nonneg(x(t) - mu)
}

/// The variance of a real-valued random variable under a discrete pmf:
/// the expectation of the squared deviation from the mean.
define discrete_variance[T](pmf: DiscretePMF[T], x: T -> Real) -> Real {
    discrete_expectation(pmf, square_dev(x, discrete_expectation(pmf, x)))
}

/// The constant real-valued function on `T`.
define const_real_fn[T](c: Real, t: T) -> Real {
    c
}

lemma const_real_fn_apply[T](c: Real, t: T) {
    const_real_fn[T](c, t) = c
}

/// The expectation of a constant function under any discrete pmf is the constant.
theorem discrete_expectation_const[T](pmf: DiscretePMF[T], c: Real) {
    discrete_expectation(pmf, const_real_fn[T](c)) = c
} by {
    let f = mass_weighted_value(pmf.mass, const_real_fn[T](c))
    forall(t: T) {
        f(t) = pmf.mass(t) * const_real_fn[T](c, t)
        f(t) = c * pmf.mass(t)
        f(t) = mul_fn(c, pmf.mass, t)
    }
    function_extensionality[T, Real](f, mul_fn(c, pmf.mass))
    finite_set_sum_scalar_mul[T, Real](c, pmf.support, pmf.mass)
    discrete_pmf_total_mass(pmf)
}


/// On an event, the mass-weighted event indicator is the point mass.
theorem mass_weighted_indicator_of_contains[T](pmf: DiscretePMF[T], event: FiniteSet[T], x: T) {
    event.contains(x) implies mass_weighted_value(pmf.mass, finite_event_indicator(event), x) = pmf.mass(x)
} by {
    if event.contains(x) {
        finite_event_indicator_one_of_contains(event, x)
        pmf.mass(x) * Real.1 = pmf.mass(x)
        mass_weighted_value(pmf.mass, finite_event_indicator(event), x) = pmf.mass(x)
    }
}

/// Off an event, the mass-weighted event indicator is zero.
theorem mass_weighted_indicator_of_not_contains[T](pmf: DiscretePMF[T], event: FiniteSet[T], x: T) {
    not event.contains(x) implies mass_weighted_value(pmf.mass, finite_event_indicator(event), x) = Real.0
} by {
    if not event.contains(x) {
        finite_event_indicator_zero_of_not_contains(event, x)
        mass_weighted_value(pmf.mass, finite_event_indicator(event), x) = Real.0
    }
}

lemma list_sum_map_eq_of_contains[T](items: List[T], f: T -> Real, g: T -> Real) {
    (forall(x: T) { items.contains(x) implies f(x) = g(x) }) implies
        sum[Real](map(items, f)) = sum[Real](map(items, g))
} by {
    if forall(x: T) { items.contains(x) implies f(x) = g(x) } {
        list_map_eq_of_contains[T, Real](items, f, g)
        sum[Real](map(items, f)) = sum[Real](map(items, g))
    }
}

lemma finite_set_sum_eq_of_contains_eq[T](s: FiniteSet[T], f: T -> Real, g: T -> Real) {
    (forall(x: T) { s.contains(x) implies f(x) = g(x) }) implies
        finite_set_sum(s, f) = finite_set_sum(s, g)
} by {
    if forall(x: T) { s.contains(x) implies f(x) = g(x) } {
        finite_set_has_unique_list(s)
        let items: List[T] satisfy {
            fs_from_list(items) = s and items.is_unique
        }
        forall(x: T) {
            if items.contains(x) {
                fs_from_list(items).underlying_set = list_set(items)
                list_set_contains_eq(items, x)
                s.contains(x)
                f(x) = g(x)
            }
        }
        forall(x: T) { items.contains(x) implies f(x) = g(x) }
        list_sum_map_eq_of_contains(items, f, g)
        sum[Real](map(items, f)) = sum[Real](map(items, g))
        finite_set_sum_eq_list_sum[T, Real](items, f)
        finite_set_sum_eq_list_sum[T, Real](items, g)
        finite_set_sum(s, f) = sum[Real](map(items, f))
        finite_set_sum(s, g) = sum[Real](map(items, g))
        finite_set_sum(s, f) = finite_set_sum(s, g)
    }
}

lemma finite_set_sum_const_zero[T](s: FiniteSet[T]) {
    finite_set_sum(s, const_real_fn[T](Real.0)) = Real.0
} by {
    let one_fn = const_real_fn[T](Real.1)
    let zero_scaled = mul_fn(Real.0, one_fn)
    finite_set_sum_scalar_mul[T, Real](Real.0, s, one_fn)
    Real.0 * finite_set_sum(s, one_fn) = finite_set_sum(s, zero_scaled)
    Real.0 * finite_set_sum(s, one_fn) = Real.0
    finite_set_sum(s, zero_scaled) = Real.0
    forall(x: T) {
        zero_scaled(x) = Real.0 * one_fn(x)
        zero_scaled(x) = const_real_fn[T](Real.0, x)
    }
    function_extensionality[T, Real](zero_scaled, const_real_fn[T](Real.0))
}

lemma finite_set_sum_zero_of_contains[T](s: FiniteSet[T], f: T -> Real) {
    (forall(x: T) { s.contains(x) implies f(x) = Real.0 }) implies finite_set_sum(s, f) = Real.0
} by {
    if forall(x: T) { s.contains(x) implies f(x) = Real.0 } {
        finite_set_sum_eq_of_contains_eq(s, f, const_real_fn[T](Real.0))
        finite_set_sum(s, f) = finite_set_sum(s, const_real_fn[T](Real.0))
        finite_set_sum_const_zero(s)
        finite_set_sum(s, f) = Real.0
    }
}

/// Summing mass over the support with an event indicator restricts to the support-event intersection.
theorem finite_set_sum_mass_indicator_support_eq_intersection[T](pmf: DiscretePMF[T], event: FiniteSet[T]) {
    finite_set_sum(pmf.support, mass_weighted_value(pmf.mass, finite_event_indicator(event))) =
        finite_set_sum(fs_intersection(pmf.support, event), pmf.mass)
} by {
    let indicator_mass = mass_weighted_value(pmf.mass, finite_event_indicator(event))
    let inside = fs_intersection(pmf.support, event)
    let outside = fs_difference(pmf.support, event)

    finite_set_intersection_union_difference_is_self[T](pmf.support, event)
    finite_set_intersection_is_disjoint_difference[T](pmf.support, event)
    finite_set_sum_disjoint_union[T, Real](inside, outside, indicator_mass)
    finite_set_sum(fs_union(inside, outside), indicator_mass) =
        finite_set_sum(inside, indicator_mass) + finite_set_sum(outside, indicator_mass)

    forall(x: T) {
        if inside.contains(x) {
            inside.underlying_set = pmf.support.underlying_set.intersection(event.underlying_set)
            intersection_contains_right(pmf.support.underlying_set, event.underlying_set, x)
            event.contains(x) = event.underlying_set.contains(x)
            event.contains(x)
            mass_weighted_indicator_of_contains(pmf, event, x)
            indicator_mass(x) = pmf.mass(x)
        }
    }
    forall(x: T) { inside.contains(x) implies indicator_mass(x) = pmf.mass(x) }
    finite_set_sum_eq_of_contains_eq(inside, indicator_mass, pmf.mass)
    finite_set_sum(inside, indicator_mass) = finite_set_sum(inside, pmf.mass)

    forall(x: T) {
        if outside.contains(x) {
            outside.underlying_set = pmf.support.underlying_set.difference(event.underlying_set)
            difference_contains_not_right(pmf.support.underlying_set, event.underlying_set, x)
            event.contains(x) = event.underlying_set.contains(x)
            not event.contains(x)
            mass_weighted_indicator_of_not_contains(pmf, event, x)
            indicator_mass(x) = Real.0
        }
    }
    forall(x: T) { outside.contains(x) implies indicator_mass(x) = Real.0 }
    finite_set_sum_zero_of_contains(outside, indicator_mass)
    finite_set_sum(outside, indicator_mass) = Real.0

    finite_set_sum(inside, pmf.mass) + Real.0 = finite_set_sum(inside, pmf.mass)
}

/// The expectation of a finite-event indicator is the probability of the event restricted to the PMF support.
theorem discrete_expectation_indicator_eq_event_prob_support_intersection[T](pmf: DiscretePMF[T], event: FiniteSet[T]) {
    discrete_expectation(pmf, finite_event_indicator(event)) =
        pmf_event_prob(pmf, fs_intersection(pmf.support, event))
} by {
    finite_set_sum_mass_indicator_support_eq_intersection(pmf, event)
}

/// Event probability depends only on the part of the event inside the PMF support.
theorem pmf_event_prob_eq_support_intersection[T](pmf: DiscretePMF[T], event: FiniteSet[T]) {
    pmf_event_prob(pmf, event) = pmf_event_prob(pmf, fs_intersection(pmf.support, event))
} by {
    let inside = fs_intersection(event, pmf.support)
    let outside = fs_difference(event, pmf.support)

    finite_set_intersection_union_difference_is_self[T](event, pmf.support)
    finite_set_intersection_is_disjoint_difference[T](event, pmf.support)
    finite_set_sum_disjoint_union[T, Real](inside, outside, pmf.mass)
    finite_set_sum(fs_union(inside, outside), pmf.mass) =
        finite_set_sum(inside, pmf.mass) + finite_set_sum(outside, pmf.mass)

    forall(x: T) {
        if outside.contains(x) {
            finite_set_difference_contains_eq[T](event, pmf.support, x)
            discrete_pmf_mass_zero_outside[T](pmf, x)
            pmf.mass(x) = Real.0
        }
    }
    forall(x: T) { outside.contains(x) implies pmf.mass(x) = Real.0 }
    finite_set_sum_zero_of_contains(outside, pmf.mass)
    finite_set_sum(outside, pmf.mass) = Real.0


    pmf_event_prob(pmf, event) = pmf_event_prob(pmf, inside)

    finite_set_intersection_comm[T](pmf.support, event)
}

/// The expectation of a finite-event indicator is the probability of that event.
theorem discrete_expectation_indicator_eq_event_prob[T](pmf: DiscretePMF[T], event: FiniteSet[T]) {
    discrete_expectation(pmf, finite_event_indicator(event)) = pmf_event_prob(pmf, event)
} by {
    discrete_expectation_indicator_eq_event_prob_support_intersection(pmf, event)
    pmf_event_prob_eq_support_intersection(pmf, event)
}

lemma finite_set_support_intersection_eq_of_contains_eq[T](support: FiniteSet[T], a: FiniteSet[T], b: FiniteSet[T]) {
    (forall(x: T) { support.contains(x) implies a.contains(x) = b.contains(x) }) implies
        fs_intersection(support, a) = fs_intersection(support, b)
} by {
    if forall(x: T) { support.contains(x) implies a.contains(x) = b.contains(x) } {
        forall(x: T) {
            finite_set_intersection_contains_eq[T](support, a, x)
            finite_set_intersection_contains_eq[T](support, b, x)
            if support.contains(x) {
                a.contains(x) = b.contains(x)
                fs_intersection(support, a).contains(x) = fs_intersection(support, b).contains(x)
            }
            if not support.contains(x) {
                fs_intersection(support, a).contains(x) = fs_intersection(support, b).contains(x)
            }
            fs_intersection(support, a).contains(x) = fs_intersection(support, b).contains(x)
            fs_intersection(support, a).underlying_set.contains(x) =
                fs_intersection(support, b).underlying_set.contains(x)
        }
        set_ext(fs_intersection(support, a).underlying_set, fs_intersection(support, b).underlying_set)
        finite_set_eq_of_underlying_set_eq(fs_intersection(support, a), fs_intersection(support, b))
        fs_intersection(support, a) = fs_intersection(support, b)
    }
}

/// Event probability is extensional for events that agree on the PMF support.
theorem pmf_event_prob_eq_of_support_contains_eq[T](pmf: DiscretePMF[T], a: FiniteSet[T], b: FiniteSet[T]) {
    (forall(x: T) { pmf.support.contains(x) implies a.contains(x) = b.contains(x) }) implies
        pmf_event_prob(pmf, a) = pmf_event_prob(pmf, b)
} by {
    if forall(x: T) { pmf.support.contains(x) implies a.contains(x) = b.contains(x) } {
        finite_set_support_intersection_eq_of_contains_eq(pmf.support, a, b)
        pmf_event_prob_eq_support_intersection(pmf, a)
        pmf_event_prob_eq_support_intersection(pmf, b)
        pmf_event_prob(pmf, fs_intersection(pmf.support, a)) =
            pmf_event_prob(pmf, fs_intersection(pmf.support, b))
        pmf_event_prob(pmf, a) = pmf_event_prob(pmf, b)
    }
}

/// Indicator expectations are extensional for events that agree on the PMF support.
theorem discrete_expectation_indicator_eq_of_support_contains_eq[T](pmf: DiscretePMF[T], a: FiniteSet[T], b: FiniteSet[T]) {
    (forall(x: T) { pmf.support.contains(x) implies a.contains(x) = b.contains(x) }) implies
        discrete_expectation(pmf, finite_event_indicator(a)) = discrete_expectation(pmf, finite_event_indicator(b))
} by {
    if forall(x: T) { pmf.support.contains(x) implies a.contains(x) = b.contains(x) } {
        discrete_expectation_indicator_eq_event_prob(pmf, a)
        discrete_expectation_indicator_eq_event_prob(pmf, b)
        pmf_event_prob_eq_of_support_contains_eq(pmf, a, b)
        pmf_event_prob(pmf, a) = pmf_event_prob(pmf, b)
        discrete_expectation(pmf, finite_event_indicator(a)) = discrete_expectation(pmf, finite_event_indicator(b))
    }
}

/// The expectation of an indicator is unchanged by restricting the event to the PMF support.
theorem discrete_expectation_indicator_eq_support_intersection_event[T](pmf: DiscretePMF[T], event: FiniteSet[T]) {
    discrete_expectation(pmf, finite_event_indicator(event)) =
        discrete_expectation(pmf, finite_event_indicator(fs_intersection(pmf.support, event)))
} by {
    forall(x: T) {
        if pmf.support.contains(x) {
            finite_set_intersection_contains_eq[T](pmf.support, event, x)
            fs_intersection(pmf.support, event).contains(x) = event.contains(x)
        }
    }
    forall(x: T) {
        pmf.support.contains(x) implies event.contains(x) = fs_intersection(pmf.support, event).contains(x)
    }
    discrete_expectation_indicator_eq_of_support_contains_eq(pmf, event, fs_intersection(pmf.support, event))
}

/// The probability of the empty event is zero.
theorem pmf_event_prob_empty[T](pmf: DiscretePMF[T]) {
    pmf_event_prob(pmf, fs_empty[T](Set[T].empty_set)) = Real.0
} by {
    finite_set_sum_empty[T, Real](pmf.mass)
}

/// The probability of the entire support is one.
theorem pmf_event_prob_support[T](pmf: DiscretePMF[T]) {
    pmf_event_prob(pmf, pmf.support) = Real.1
} by {
    discrete_pmf_total_mass(pmf)
}

/// Event probabilities are nonnegative.
theorem pmf_event_prob_nonneg[T](pmf: DiscretePMF[T], event: FiniteSet[T]) {
    Real.0 <= pmf_event_prob(pmf, event)
} by {
    forall(x: T) {
        discrete_pmf_mass_nonneg(pmf, x)
        Real.0 <= pmf.mass(x)
    }
    finite_set_sum_nonneg[T](event, pmf.mass)
    Real.0 <= finite_set_sum(event, pmf.mass)
}

/// The expectation is monotone under pointwise order on the support.
theorem discrete_expectation_monotone[T](pmf: DiscretePMF[T], x: T -> Real, y: T -> Real) {
    (forall(t: T) { pmf.support.contains(t) implies x(t) <= y(t) }) implies
        discrete_expectation(pmf, x) <= discrete_expectation(pmf, y)
} by {
    if forall(t: T) { pmf.support.contains(t) implies x(t) <= y(t) } {
        let x_weighted = mass_weighted_value(pmf.mass, x)
        let y_weighted = mass_weighted_value(pmf.mass, y)
        forall(t: T) {
            if pmf.support.contains(t) {
                x(t) <= y(t)
                discrete_pmf_mass_nonneg(pmf, t)
                pmf.mass(t) * x(t) <= pmf.mass(t) * y(t)
                x_weighted(t) = pmf.mass(t) * x(t)
                y_weighted(t) = pmf.mass(t) * y(t)
                x_weighted(t) <= y_weighted(t)
            }
            if not pmf.support.contains(t) {
                discrete_pmf_mass_zero_outside(pmf, t)
                x_weighted(t) = pmf.mass(t) * x(t)
                y_weighted(t) = pmf.mass(t) * y(t)
                Real.0 * x(t) = Real.0
                Real.0 * y(t) = Real.0
                x_weighted(t) <= y_weighted(t)
            }
            pmf.support.contains(t) or not pmf.support.contains(t)
            x_weighted(t) <= y_weighted(t)
        }
        pointwise_le[T](x_weighted, y_weighted)
        finite_set_sum_le[T](pmf.support, x_weighted, y_weighted)
        finite_set_sum(pmf.support, x_weighted) <= finite_set_sum(pmf.support, y_weighted)
        discrete_expectation(pmf, x) <= discrete_expectation(pmf, y)
    }
}

/// If a random variable is bounded above by a constant on the support, then
/// its expectation is bounded above by that constant.
theorem discrete_expectation_le_const[T](pmf: DiscretePMF[T], x: T -> Real, c: Real) {
    (forall(t: T) { pmf.support.contains(t) implies x(t) <= c }) implies
        discrete_expectation(pmf, x) <= c
} by {
    if forall(t: T) { pmf.support.contains(t) implies x(t) <= c } {
        let c_fn = const_real_fn[T](c)
        forall(t: T) {
            if pmf.support.contains(t) {
                const_real_fn_apply[T](c, t)
                x(t) <= c_fn(t)
            }
        }
        forall(t: T) { pmf.support.contains(t) implies x(t) <= c_fn(t) }
        discrete_expectation_monotone(pmf, x, c_fn)
        discrete_expectation(pmf, x) <= discrete_expectation(pmf, c_fn)
        discrete_expectation_const(pmf, c)
        discrete_expectation(pmf, x) <= c
    }
}

/// If a random variable is bounded below by a constant on the support, then
/// its expectation is bounded below by that constant.
theorem const_le_discrete_expectation[T](pmf: DiscretePMF[T], x: T -> Real, c: Real) {
    (forall(t: T) { pmf.support.contains(t) implies c <= x(t) }) implies
        c <= discrete_expectation(pmf, x)
} by {
    if forall(t: T) { pmf.support.contains(t) implies c <= x(t) } {
        let c_fn = const_real_fn[T](c)
        forall(t: T) {
            if pmf.support.contains(t) {
                const_real_fn_apply[T](c, t)
                c_fn(t) <= x(t)
            }
        }
        forall(t: T) { pmf.support.contains(t) implies c_fn(t) <= x(t) }
        discrete_expectation_monotone(pmf, c_fn, x)
        discrete_expectation(pmf, c_fn) <= discrete_expectation(pmf, x)
        discrete_expectation_const(pmf, c)
        c <= discrete_expectation(pmf, x)
    }
}

/// A nonnegative random variable on the support has nonnegative expectation.
theorem discrete_expectation_nonneg[T](pmf: DiscretePMF[T], x: T -> Real) {
    (forall(t: T) { pmf.support.contains(t) implies Real.0 <= x(t) }) implies
        Real.0 <= discrete_expectation(pmf, x)
} by {
    if forall(t: T) { pmf.support.contains(t) implies Real.0 <= x(t) } {
        const_le_discrete_expectation(pmf, x, Real.0)
        Real.0 <= discrete_expectation(pmf, x)
    }
}

/// Variance is nonnegative because it is the expectation of a squared deviation.
theorem discrete_variance_nonneg[T](pmf: DiscretePMF[T], x: T -> Real) {
    Real.0 <= discrete_variance(pmf, x)
} by {
    let mu = discrete_expectation(pmf, x)
    let dev = square_dev(x, mu)
    forall(t: T) {
        if pmf.support.contains(t) {
            square_dev_nonneg(x, mu, t)
            Real.0 <= dev(t)
        }
    }
    discrete_expectation_nonneg(pmf, dev)
    Real.0 <= discrete_expectation(pmf, dev)
}

/// The expectation of a finite-event indicator is nonnegative.
theorem discrete_expectation_indicator_nonneg[T](pmf: DiscretePMF[T], event: FiniteSet[T]) {
    Real.0 <= discrete_expectation(pmf, finite_event_indicator(event))
} by {
    forall(t: T) {
        if pmf.support.contains(t) {
            finite_event_indicator_nonneg(event, t)
        }
    }
    discrete_expectation_nonneg(pmf, finite_event_indicator(event))
}

/// The expectation of a finite-event indicator is at most one.
theorem discrete_expectation_indicator_le_one[T](pmf: DiscretePMF[T], event: FiniteSet[T]) {
    discrete_expectation(pmf, finite_event_indicator(event)) <= Real.1
} by {
    forall(t: T) {
        if pmf.support.contains(t) {
            finite_event_indicator_le_one(event, t)
        }
    }
    discrete_expectation_le_const(pmf, finite_event_indicator(event), Real.1)
}

/// The expectation of an event indicator lies between zero and one.
theorem discrete_expectation_indicator_bounds[T](pmf: DiscretePMF[T], event: FiniteSet[T]) {
    Real.0 <= discrete_expectation(pmf, finite_event_indicator(event)) and
        discrete_expectation(pmf, finite_event_indicator(event)) <= Real.1
} by {
    discrete_expectation_indicator_nonneg(pmf, event)
    discrete_expectation_indicator_le_one(pmf, event)
}

/// The probability of a disjoint union of events is the sum of their probabilities.
theorem pmf_event_prob_disjoint_union[T](pmf: DiscretePMF[T], a: FiniteSet[T], b: FiniteSet[T]) {
    a.is_disjoint(b) implies
        pmf_event_prob(pmf, fs_union(a, b)) = pmf_event_prob(pmf, a) + pmf_event_prob(pmf, b)
} by {
    if a.is_disjoint(b) {
        finite_set_sum_disjoint_union[T, Real](a, b, pmf.mass)
    }
}

/// Inclusion-exclusion for two discrete events:
/// the probability of the union plus the probability of the intersection
/// equals the sum of the individual probabilities.
theorem pmf_event_prob_inclusion_exclusion[T](pmf: DiscretePMF[T], a: FiniteSet[T], b: FiniteSet[T]) {
    pmf_event_prob(pmf, fs_union(a, b)) + pmf_event_prob(pmf, fs_intersection(a, b))
        = pmf_event_prob(pmf, a) + pmf_event_prob(pmf, b)
} by {
    // A ∪ B = A ∪ (B \ A), with A and (B \ A) disjoint.
    union_with_difference_decomp_rev(a.underlying_set, b.underlying_set)
    fs_union(a, b).underlying_set = a.underlying_set.union(b.underlying_set)
    fs_difference(b, a).underlying_set = b.underlying_set.difference(a.underlying_set)
    fs_union(a, fs_difference(b, a)).underlying_set =
        a.underlying_set.union(fs_difference(b, a).underlying_set)
    finite_set_ext(fs_union(a, b), fs_union(a, fs_difference(b, a)))
    fs_union(a, b) = fs_union(a, fs_difference(b, a))

    a.underlying_set.is_disjoint(b.underlying_set.difference(a.underlying_set))
    a.is_disjoint(fs_difference(b, a))

    pmf_event_prob_disjoint_union[T](pmf, a, fs_difference(b, a))
    pmf_event_prob(pmf, fs_union(a, b)) =
        pmf_event_prob(pmf, a) + pmf_event_prob(pmf, fs_difference(b, a))

    // B = (B ∩ A) ∪ (B \ A), disjointly.
    finite_set_intersection_union_difference_is_self[T](b, a)
    finite_set_intersection_is_disjoint_difference[T](b, a)

    pmf_event_prob_disjoint_union[T](pmf, fs_intersection(b, a), fs_difference(b, a))

    // Commute the intersection.
    intersection_comm(a.underlying_set, b.underlying_set)
    finite_set_ext(fs_intersection(a, b), fs_intersection(b, a))

    pmf_event_prob(pmf, b) =
        pmf_event_prob(pmf, fs_intersection(a, b)) + pmf_event_prob(pmf, fs_difference(b, a))

    let pa = pmf_event_prob(pmf, a)
    let pb = pmf_event_prob(pmf, b)
    let pab_union = pmf_event_prob(pmf, fs_union(a, b))
    let pab_inter = pmf_event_prob(pmf, fs_intersection(a, b))
    let pb_minus_a = pmf_event_prob(pmf, fs_difference(b, a))
    pa + pb = (pa + pb_minus_a) + pab_inter
}

/// General union law for two discrete events:
/// `P(A ∪ B) = P(A) + P(B) - P(A ∩ B)`.
theorem pmf_event_prob_union[T](pmf: DiscretePMF[T], a: FiniteSet[T], b: FiniteSet[T]) {
    pmf_event_prob(pmf, fs_union(a, b))
        = pmf_event_prob(pmf, a) + pmf_event_prob(pmf, b) - pmf_event_prob(pmf, fs_intersection(a, b))
} by {
    pmf_event_prob_inclusion_exclusion(pmf, a, b)
    let pa = pmf_event_prob(pmf, a)
    let pb = pmf_event_prob(pmf, b)
    let pu = pmf_event_prob(pmf, fs_union(a, b))
    let pi = pmf_event_prob(pmf, fs_intersection(a, b))
    pu + pi = pa + pb
}

/// The support of a PMF decomposes as the disjoint union of any event's
/// intersection with the support and the support minus the event, so
/// `P(A ∩ support) + P(support \ A) = 1`.
theorem pmf_event_prob_support_split[T](pmf: DiscretePMF[T], a: FiniteSet[T]) {
    pmf_event_prob(pmf, fs_intersection(pmf.support, a))
        + pmf_event_prob(pmf, fs_difference(pmf.support, a)) = Real.1
} by {
    finite_set_intersection_union_difference_is_self[T](pmf.support, a)
    fs_union(fs_intersection(pmf.support, a), fs_difference(pmf.support, a)) = pmf.support
    finite_set_intersection_is_disjoint_difference[T](pmf.support, a)
    fs_intersection(pmf.support, a).is_disjoint(fs_difference(pmf.support, a))
    pmf_event_prob_disjoint_union[T](pmf, fs_intersection(pmf.support, a), fs_difference(pmf.support, a))
    let pi = pmf_event_prob(pmf, fs_intersection(pmf.support, a))
    let pd = pmf_event_prob(pmf, fs_difference(pmf.support, a))
    let pu = pmf_event_prob(pmf, fs_union(fs_intersection(pmf.support, a), fs_difference(pmf.support, a)))
    pmf_event_prob(pmf, pmf.support) = pi + pd
    pmf_event_prob_support[T](pmf)
}

lemma finite_set_intersection_eq_right_of_subset[T](left: FiniteSet[T], right: FiniteSet[T]) {
    right.subset_eq(left) implies fs_intersection(left, right) = right
} by {
    if right.subset_eq(left) {
        intersection_with_superset_is_self(right.underlying_set, left.underlying_set)
        right.underlying_set.intersection(left.underlying_set) = right.underlying_set
        intersection_comm(left.underlying_set, right.underlying_set)
        fs_intersection(left, right).underlying_set =
            left.underlying_set.intersection(right.underlying_set)
        finite_set_ext(fs_intersection(left, right), right)
        fs_intersection(left, right) = right
    }
}

lemma real_le_of_add_nonneg_eq(a: Real, b: Real, c: Real) {
    Real.0 <= b and a + b = c implies a <= c
} by {
    if Real.0 <= b and a + b = c {
        add_le_add_left[Real](Real.0, b, a)
        Real.add_identity_right(a)
        a <= c
    }
}

/// The probability of a union of two finite events is at most the sum of their probabilities.
theorem pmf_event_prob_union_le_sum[T](pmf: DiscretePMF[T], a: FiniteSet[T], b: FiniteSet[T]) {
    pmf_event_prob(pmf, fs_union(a, b)) <= pmf_event_prob(pmf, a) + pmf_event_prob(pmf, b)
} by {
    pmf_event_prob_inclusion_exclusion[T](pmf, a, b)
    let pu = pmf_event_prob(pmf, fs_union(a, b))
    let pi = pmf_event_prob(pmf, fs_intersection(a, b))
    let pa = pmf_event_prob(pmf, a)
    let pb = pmf_event_prob(pmf, b)
    pu + pi = pa + pb
    pmf_event_prob_nonneg[T](pmf, fs_intersection(a, b))
    Real.0 <= pi
    real_le_of_add_nonneg_eq(pu, pi, pa + pb)
    pu <= pa + pb
}

/// Event probability is monotone under inclusion of finite events.
theorem pmf_event_prob_mono[T](pmf: DiscretePMF[T], a: FiniteSet[T], b: FiniteSet[T]) {
    a.subset_eq(b) implies pmf_event_prob(pmf, a) <= pmf_event_prob(pmf, b)
} by {
    if a.subset_eq(b) {
        finite_set_intersection_eq_right_of_subset(b, a)
        finite_set_intersection_union_difference_is_self[T](b, a)
        finite_set_intersection_is_disjoint_difference[T](b, a)
        a.is_disjoint(fs_difference(b, a))
        pmf_event_prob_disjoint_union[T](pmf, a, fs_difference(b, a))
        let pa = pmf_event_prob(pmf, a)
        let pd = pmf_event_prob(pmf, fs_difference(b, a))
        let pb = pmf_event_prob(pmf, b)
        pmf_event_prob(pmf, fs_union(a, fs_difference(b, a))) = pa + pd
        pmf_event_prob_nonneg(pmf, fs_difference(b, a))
        Real.0 <= pd
        real_le_of_add_nonneg_eq(pa, pd, pb)
        pa <= pb
        pmf_event_prob(pmf, a) <= pmf_event_prob(pmf, b)
    }
}

/// An event contained in the support has probability at most one.
theorem pmf_event_prob_le_one[T](pmf: DiscretePMF[T], event: FiniteSet[T]) {
    event.subset_eq(pmf.support) implies pmf_event_prob(pmf, event) <= Real.1
} by {
    if event.subset_eq(pmf.support) {
        pmf_event_prob_mono(pmf, event, pmf.support)
        pmf_event_prob_support[T](pmf)
        pmf_event_prob(pmf, event) <= Real.1
    }
}

/// The probability of an intersection is at most the probability of its left event.
theorem pmf_event_prob_intersection_le_left[T](pmf: DiscretePMF[T], a: FiniteSet[T], b: FiniteSet[T]) {
    pmf_event_prob(pmf, fs_intersection(a, b)) <= pmf_event_prob(pmf, a)
} by {
    let inter = fs_intersection(a, b)
    finite_set_subset_refl[T](inter)
    inter.subset_eq(inter)
    finite_set_subset_intersection_eq[T](a, b, inter)
    inter.subset_eq(inter) = (inter.subset_eq(a) and inter.subset_eq(b))
    inter.subset_eq(a)
    pmf_event_prob_mono[T](pmf, inter, a)
    pmf_event_prob(pmf, inter) <= pmf_event_prob(pmf, a)
}

/// The probability of an intersection is at most the probability of its right event.
theorem pmf_event_prob_intersection_le_right[T](pmf: DiscretePMF[T], a: FiniteSet[T], b: FiniteSet[T]) {
    pmf_event_prob(pmf, fs_intersection(a, b)) <= pmf_event_prob(pmf, b)
} by {
    let inter = fs_intersection(a, b)
    finite_set_subset_refl[T](inter)
    inter.subset_eq(inter)
    finite_set_subset_intersection_eq[T](a, b, inter)
    inter.subset_eq(inter) = (inter.subset_eq(a) and inter.subset_eq(b))
    inter.subset_eq(b)
    pmf_event_prob_mono[T](pmf, inter, b)
    pmf_event_prob(pmf, inter) <= pmf_event_prob(pmf, b)
}

/// The variance formula: `Var(X) = E(X^2) - E(X)^2` for any discrete pmf.
theorem discrete_variance_formula[T](pmf: DiscretePMF[T], x: T -> Real) {
    discrete_variance(pmf, x) = discrete_expectation(pmf, square_fn(x))
        - discrete_expectation(pmf, x) * discrete_expectation(pmf, x)
} by {
    let mu = discrete_expectation(pmf, x)
    let g = add_fn(mul_fn(-((mu + mu)), x), const_real_fn[T](mu * mu))
    let decomp = add_fn(square_fn(x), g)
    forall(t: T) {
        let a = x(t)
        let b = mu
        square_dev(x, mu, t) = (a - b) * (a - b)
        (a - b) * (a - b) = (a - b) * a - (a - b) * b
        (a - b) * a = a * a - b * a
        (a - b) * b = a * b - b * b
        (a - b) * (a - b) = a * a - b * a - (a * b - b * b)
        a * b = b * a
        (a - b) * (a - b) = a * a - b * a - (b * a - b * b)
        -(b * a - b * b) = -(b * a) + b * b
        -(b * a) = - b * a
        -(b * a - b * b) = -(b * a) + b * b
        a * a - b * a - (b * a - b * b) = a * a - b * a + (-(b * a) + b * b)
        a * a - b * a + (-(b * a) + b * b) = a * a - b * a + -(b * a) + b * b
        a * a - b * a + -(b * a) + b * b = a * a - b * a - b * a + b * b
        a * a - b * a - (b * a - b * b) = a * a - b * a - b * a + b * b
        (a - b) * (a - b) = a * a - b * a - b * a + b * b
        b * a + b * a = (b + b) * a
        a * a - b * a - b * a + b * b = a * a - (b * a + b * a) + b * b
        a * a - (b * a + b * a) + b * b = a * a - (b + b) * a + b * b
        a * a - (b + b) * a + b * b = a * a + -((b + b) * a) + b * b
        -((b + b) * a) = -(b + b) * a
        a * a + -((b + b) * a) + b * b = a * a + -(b + b) * a + b * b
        (a - b) * (a - b) = a * a + -(b + b) * a + b * b
        square_fn(x, t) = a * a
        mul_fn(-(b + b), x, t) = -(b + b) * a
        const_real_fn[T](b * b, t) = b * b
        add_fn(mul_fn(-(b + b), x), const_real_fn[T](b * b), t) =
            mul_fn(-(b + b), x, t) + const_real_fn[T](b * b, t)
        g(t) = -(b + b) * a + b * b
        add_fn(square_fn(x), g, t) = square_fn(x, t) + g(t)
        decomp(t) = a * a + (-(b + b) * a + b * b)
        a * a + (-(b + b) * a + b * b) = a * a + -(b + b) * a + b * b
        decomp(t) = a * a + -(b + b) * a + b * b
        square_dev(x, mu, t) = decomp(t)
    }
    function_extensionality[T, Real](square_dev(x, mu), decomp)
    square_dev(x, mu) = decomp
    discrete_variance(pmf, x) = discrete_expectation(pmf, square_dev(x, mu))
    discrete_variance(pmf, x) = discrete_expectation(pmf, decomp)
    discrete_expectation_add[T](pmf, square_fn(x), g)
    discrete_expectation(pmf, decomp) =
        discrete_expectation(pmf, square_fn(x)) + discrete_expectation(pmf, g)
    discrete_expectation_add[T](pmf, mul_fn(-((mu + mu)), x), const_real_fn[T](mu * mu))
    discrete_expectation(pmf, g) =
        discrete_expectation(pmf, mul_fn(-((mu + mu)), x)) +
            discrete_expectation(pmf, const_real_fn[T](mu * mu))
    discrete_expectation_scalar_mul[T](pmf, -((mu + mu)), x)
    discrete_expectation(pmf, mul_fn(-((mu + mu)), x)) = -((mu + mu)) * mu
    discrete_expectation_const[T](pmf, mu * mu)
    discrete_expectation(pmf, const_real_fn[T](mu * mu)) = mu * mu
    discrete_expectation(pmf, g) = -((mu + mu)) * mu + mu * mu
    (mu + mu) * mu = mu * mu + mu * mu
    -((mu + mu) * mu) = -(mu * mu + mu * mu)
    -(mu * mu + mu * mu) = -(mu * mu) + -(mu * mu)
    -((mu + mu) * mu) = -(mu * mu) + -(mu * mu)
    -((mu + mu)) * mu = -((mu + mu) * mu)
    -((mu + mu)) * mu + mu * mu = -(mu * mu) + -(mu * mu) + mu * mu
    -(mu * mu) + mu * mu = Real.0
    -(mu * mu) + -(mu * mu) + mu * mu = -(mu * mu) + (-(mu * mu) + mu * mu)
    -(mu * mu) + (-(mu * mu) + mu * mu) = -(mu * mu) + Real.0
    -(mu * mu) + Real.0 = -(mu * mu)
    -(mu * mu) + -(mu * mu) + mu * mu = -(mu * mu)
    -((mu + mu)) * mu + mu * mu = -(mu * mu)
    discrete_expectation(pmf, g) = -(mu * mu)
    discrete_expectation(pmf, decomp) =
        discrete_expectation(pmf, square_fn(x)) + -(mu * mu)
    discrete_expectation(pmf, decomp) =
        discrete_expectation(pmf, square_fn(x)) - mu * mu
}

/// Expectation negates with the random variable: `E(-X) = -E(X)`.
theorem discrete_expectation_neg[T](pmf: DiscretePMF[T], x: T -> Real) {
    discrete_expectation(pmf, mul_fn(-Real.1, x)) = -discrete_expectation(pmf, x)
} by {
    discrete_expectation_scalar_mul[T](pmf, -Real.1, x)
    -Real.1 * discrete_expectation(pmf, x) = -discrete_expectation(pmf, x)
}

/// Expectation distributes over differences: `E(X + (-1) * Y) = E(X) - E(Y)`.
/// The difference is expressed via the existing pointwise additive and scalar-multiplied
/// random-variable combinators because `acornlib` does not yet have a dedicated
/// `sub_fn` helper.
theorem discrete_expectation_sub[T](pmf: DiscretePMF[T], x: T -> Real, y: T -> Real) {
    discrete_expectation(pmf, add_fn(x, mul_fn(-Real.1, y)))
        = discrete_expectation(pmf, x) - discrete_expectation(pmf, y)
} by {
    discrete_expectation_add[T](pmf, x, mul_fn(-Real.1, y))
    discrete_expectation(pmf, add_fn(x, mul_fn(-Real.1, y))) = discrete_expectation(pmf, x) + discrete_expectation(pmf, mul_fn(-Real.1, y))
    discrete_expectation_neg[T](pmf, y)
    discrete_expectation(pmf, mul_fn(-Real.1, y)) = -discrete_expectation(pmf, y)
    discrete_expectation(pmf, x) + -discrete_expectation(pmf, y) = discrete_expectation(pmf, x) - discrete_expectation(pmf, y)
}

/// Expectation is linear in a two-term combination:
/// `E(a * X + b * Y) = a * E(X) + b * E(Y)` for real constants `a` and `b`.
theorem discrete_expectation_linear_combo[T](pmf: DiscretePMF[T], a: Real, x: T -> Real, b: Real, y: T -> Real) {
    discrete_expectation(pmf, add_fn(mul_fn(a, x), mul_fn(b, y)))
        = a * discrete_expectation(pmf, x) + b * discrete_expectation(pmf, y)
} by {
    discrete_expectation_add[T](pmf, mul_fn(a, x), mul_fn(b, y))
    discrete_expectation(pmf, add_fn(mul_fn(a, x), mul_fn(b, y))) = discrete_expectation(pmf, mul_fn(a, x)) + discrete_expectation(pmf, mul_fn(b, y))
    discrete_expectation_scalar_mul[T](pmf, a, x)
    discrete_expectation(pmf, mul_fn(a, x)) = a * discrete_expectation(pmf, x)
    discrete_expectation_scalar_mul[T](pmf, b, y)
    discrete_expectation(pmf, mul_fn(b, y)) = b * discrete_expectation(pmf, y)
}

/// Variance is invariant under negation of the random variable: `Var(-X) = Var(X)`.
theorem discrete_variance_neg[T](pmf: DiscretePMF[T], x: T -> Real) {
    discrete_variance(pmf, mul_fn(-Real.1, x)) = discrete_variance(pmf, x)
} by {
    let neg_x = mul_fn(-Real.1, x)
    forall(t: T) {
        mul_fn(-Real.1, x, t) = -Real.1 * x(t)
        neg_x(t) = -Real.1 * x(t)
        -Real.1 * x(t) = -x(t)
        neg_x(t) = -x(t)
        square_fn(neg_x, t) = neg_x(t) * neg_x(t)
        mul_neg_neg[Real](x(t), x(t))
        -x(t) * -x(t) = x(t) * x(t)
        neg_x(t) * neg_x(t) = x(t) * x(t)
        square_fn(x, t) = x(t) * x(t)
        square_fn(neg_x, t) = square_fn(x, t)
    }
    function_extensionality[T, Real](square_fn(neg_x), square_fn(x))
    square_fn(neg_x) = square_fn(x)
    discrete_expectation(pmf, square_fn(neg_x)) = discrete_expectation(pmf, square_fn(x))
    discrete_variance_formula[T](pmf, neg_x)
    discrete_variance_formula[T](pmf, x)
    discrete_expectation_neg[T](pmf, x)
    discrete_expectation(pmf, neg_x) = -discrete_expectation(pmf, x)
    let mu = discrete_expectation(pmf, x)
    discrete_expectation(pmf, neg_x) * discrete_expectation(pmf, neg_x) = (-mu) * (-mu)
    mul_neg_neg[Real](mu, mu)
    -mu * -mu = mu * mu
    discrete_expectation(pmf, neg_x) * discrete_expectation(pmf, neg_x) = mu * mu
    discrete_variance(pmf, neg_x) = discrete_expectation(pmf, square_fn(neg_x)) - discrete_expectation(pmf, neg_x) * discrete_expectation(pmf, neg_x)
    discrete_variance(pmf, neg_x) = discrete_expectation(pmf, square_fn(x)) - mu * mu
    discrete_variance(pmf, x) = discrete_expectation(pmf, square_fn(x)) - mu * mu
}

/// The variance of a constant random variable is zero.
theorem discrete_variance_const[T](pmf: DiscretePMF[T], c: Real) {
    discrete_variance(pmf, const_real_fn[T](c)) = Real.0
} by {
    discrete_variance_formula[T](pmf, const_real_fn[T](c))
    discrete_expectation_const[T](pmf, c)
    discrete_expectation(pmf, const_real_fn[T](c)) = c
    forall(t: T) {
        square_fn(const_real_fn[T](c), t) = const_real_fn[T](c, t) * const_real_fn[T](c, t)
        const_real_fn_apply[T](c, t)
        const_real_fn[T](c, t) = c
        square_fn(const_real_fn[T](c), t) = c * c
        const_real_fn[T](c * c, t) = c * c
        square_fn(const_real_fn[T](c), t) = const_real_fn[T](c * c, t)
    }
    function_extensionality[T, Real](square_fn(const_real_fn[T](c)), const_real_fn[T](c * c))
    square_fn(const_real_fn[T](c)) = const_real_fn[T](c * c)
    discrete_expectation(pmf, square_fn(const_real_fn[T](c))) =
        discrete_expectation(pmf, const_real_fn[T](c * c))
    discrete_expectation_const[T](pmf, c * c)
    discrete_expectation(pmf, const_real_fn[T](c * c)) = c * c
    discrete_expectation(pmf, square_fn(const_real_fn[T](c))) = c * c
    discrete_variance(pmf, const_real_fn[T](c)) = c * c - c * c
    c * c - c * c = Real.0
}

/// The expectation of the zero function under any discrete pmf is zero.
theorem discrete_expectation_zero[T](pmf: DiscretePMF[T]) {
    discrete_expectation(pmf, const_real_fn[T](Real.0)) = Real.0
} by {
    discrete_expectation_const[T](pmf, Real.0)
}

/// The probability of `support \ a` is `1` minus the probability of `support ∩ a`.
theorem pmf_event_prob_complement[T](pmf: DiscretePMF[T], a: FiniteSet[T]) {
    pmf_event_prob(pmf, fs_difference(pmf.support, a))
        = Real.1 - pmf_event_prob(pmf, fs_intersection(pmf.support, a))
} by {
    pmf_event_prob_support_split[T](pmf, a)
}

/// The expectation of the indicator of `support \ a` is one minus the expectation of the indicator of `a`.
theorem discrete_expectation_indicator_support_complement[T](pmf: DiscretePMF[T], a: FiniteSet[T]) {
    discrete_expectation(pmf, finite_event_indicator(fs_difference(pmf.support, a))) =
        Real.1 - discrete_expectation(pmf, finite_event_indicator(a))
} by {
    discrete_expectation_indicator_eq_event_prob(pmf, fs_difference(pmf.support, a))
    discrete_expectation(pmf, finite_event_indicator(fs_difference(pmf.support, a))) =
        pmf_event_prob(pmf, fs_difference(pmf.support, a))
    pmf_event_prob_complement(pmf, a)
    pmf_event_prob(pmf, fs_difference(pmf.support, a)) =
        Real.1 - pmf_event_prob(pmf, fs_intersection(pmf.support, a))
    pmf_event_prob_eq_support_intersection(pmf, a)
    pmf_event_prob(pmf, a) = pmf_event_prob(pmf, fs_intersection(pmf.support, a))
    discrete_expectation_indicator_eq_event_prob(pmf, a)
    discrete_expectation(pmf, finite_event_indicator(a)) = pmf_event_prob(pmf, a)
    pmf_event_prob(pmf, fs_intersection(pmf.support, a)) =
        discrete_expectation(pmf, finite_event_indicator(a))
    discrete_expectation(pmf, finite_event_indicator(fs_difference(pmf.support, a))) =
        Real.1 - discrete_expectation(pmf, finite_event_indicator(a))
}

/// Two discrete events are independent under a pmf when the probability of their
/// intersection equals the product of their probabilities.
define discrete_events_independent[T](pmf: DiscretePMF[T], a: FiniteSet[T], b: FiniteSet[T]) -> Bool {
    pmf_event_prob(pmf, fs_intersection(a, b))
        = pmf_event_prob(pmf, a) * pmf_event_prob(pmf, b)
}

/// Independence of two events is symmetric.
theorem discrete_events_independent_comm[T](pmf: DiscretePMF[T], a: FiniteSet[T], b: FiniteSet[T]) {
    discrete_events_independent(pmf, a, b) implies discrete_events_independent(pmf, b, a)
} by {
    if discrete_events_independent(pmf, a, b) {
        intersection_comm(a.underlying_set, b.underlying_set)
        fs_intersection(a, b).underlying_set = a.underlying_set.intersection(b.underlying_set)
        fs_intersection(b, a).underlying_set = b.underlying_set.intersection(a.underlying_set)
        fs_intersection(a, b).underlying_set = fs_intersection(b, a).underlying_set
        finite_set_ext(fs_intersection(a, b), fs_intersection(b, a))
        fs_intersection(a, b) = fs_intersection(b, a)
        pmf_event_prob(pmf, fs_intersection(b, a)) = pmf_event_prob(pmf, fs_intersection(a, b))
    }
}

/// If `a` and `b` are independent, then `P(A ∪ B) = P(A) + P(B) - P(A) * P(B)`.
theorem discrete_events_independent_union[T](pmf: DiscretePMF[T], a: FiniteSet[T], b: FiniteSet[T]) {
    discrete_events_independent(pmf, a, b) implies
        pmf_event_prob(pmf, fs_union(a, b))
            = pmf_event_prob(pmf, a) + pmf_event_prob(pmf, b)
                - pmf_event_prob(pmf, a) * pmf_event_prob(pmf, b)
} by {
    if discrete_events_independent(pmf, a, b) {
        pmf_event_prob_union[T](pmf, a, b)
        pmf_event_prob(pmf, fs_intersection(a, b)) =
            pmf_event_prob(pmf, a) * pmf_event_prob(pmf, b)
    }
}

/// The empty event is independent of every finite event.
theorem discrete_events_independent_empty_left[T](pmf: DiscretePMF[T], event: FiniteSet[T]) {
    discrete_events_independent(pmf, FiniteSet.empty[T], event)
} by {
    finite_set_intersection_comm[T](FiniteSet.empty[T], event)
    fs_intersection(FiniteSet.empty[T], event) = fs_intersection(event, FiniteSet.empty[T])
    finite_set_intersection_empty_right[T](event)
    fs_intersection(event, FiniteSet.empty[T]) = FiniteSet.empty[T]
    fs_intersection(FiniteSet.empty[T], event) = FiniteSet.empty[T]
    pmf_event_prob_empty[T](pmf)
    pmf_event_prob(pmf, FiniteSet.empty[T]) = Real.0
    pmf_event_prob(pmf, fs_intersection(FiniteSet.empty[T], event)) = Real.0
    pmf_event_prob(pmf, FiniteSet.empty[T]) * pmf_event_prob(pmf, event) =
        Real.0 * pmf_event_prob(pmf, event)
    Real.0 * pmf_event_prob(pmf, event) = Real.0
    pmf_event_prob(pmf, fs_intersection(FiniteSet.empty[T], event)) =
        pmf_event_prob(pmf, FiniteSet.empty[T]) * pmf_event_prob(pmf, event)
    discrete_events_independent(pmf, FiniteSet.empty[T], event) =
        (pmf_event_prob(pmf, fs_intersection(FiniteSet.empty[T], event)) =
            pmf_event_prob(pmf, FiniteSet.empty[T]) * pmf_event_prob(pmf, event))
}

/// The whole PMF support is independent of every finite event.
theorem discrete_events_independent_support_left[T](pmf: DiscretePMF[T], event: FiniteSet[T]) {
    discrete_events_independent(pmf, pmf.support, event)
} by {
    pmf_event_prob_eq_support_intersection[T](pmf, event)
    pmf_event_prob(pmf, event) = pmf_event_prob(pmf, fs_intersection(pmf.support, event))
    pmf_event_prob_support[T](pmf)
    pmf_event_prob(pmf, pmf.support) = Real.1
    pmf_event_prob(pmf, pmf.support) * pmf_event_prob(pmf, event) =
        Real.1 * pmf_event_prob(pmf, event)
    Real.1 * pmf_event_prob(pmf, event) = pmf_event_prob(pmf, event)
    pmf_event_prob(pmf, pmf.support) * pmf_event_prob(pmf, event) = pmf_event_prob(pmf, event)
    pmf_event_prob(pmf, fs_intersection(pmf.support, event)) =
        pmf_event_prob(pmf, pmf.support) * pmf_event_prob(pmf, event)
    discrete_events_independent(pmf, pmf.support, event) =
        (pmf_event_prob(pmf, fs_intersection(pmf.support, event)) =
            pmf_event_prob(pmf, pmf.support) * pmf_event_prob(pmf, event))
}

/// Variance scales by the square of the constant: `Var(c * X) = c^2 * Var(X)`.
theorem discrete_variance_scalar_mul[T](pmf: DiscretePMF[T], c: Real, x: T -> Real) {
    discrete_variance(pmf, mul_fn(c, x)) = c * c * discrete_variance(pmf, x)
} by {
    let cx = mul_fn(c, x)
    forall(t: T) {
        square_fn(cx, t) = cx(t) * cx(t)
        mul_fn(c, x, t) = c * x(t)
        cx(t) = c * x(t)
        cx(t) * cx(t) = (c * x(t)) * (c * x(t))
        (c * x(t)) * (c * x(t)) = c * (x(t) * (c * x(t)))
        x(t) * (c * x(t)) = c * (x(t) * x(t))
        (c * x(t)) * (c * x(t)) = c * (c * (x(t) * x(t)))
        c * (c * (x(t) * x(t))) = (c * c) * (x(t) * x(t))
        square_fn(x, t) = x(t) * x(t)
        square_fn(cx, t) = (c * c) * square_fn(x, t)
        mul_fn(c * c, square_fn(x), t) = (c * c) * square_fn(x, t)
        square_fn(cx, t) = mul_fn(c * c, square_fn(x), t)
    }
    function_extensionality[T, Real](square_fn(cx), mul_fn(c * c, square_fn(x)))
    square_fn(cx) = mul_fn(c * c, square_fn(x))
    discrete_expectation_scalar_mul[T](pmf, c * c, square_fn(x))
    discrete_expectation(pmf, mul_fn(c * c, square_fn(x))) = (c * c) * discrete_expectation(pmf, square_fn(x))
    discrete_expectation(pmf, square_fn(cx)) = (c * c) * discrete_expectation(pmf, square_fn(x))

    discrete_expectation_scalar_mul[T](pmf, c, x)
    discrete_expectation(pmf, cx) = c * discrete_expectation(pmf, x)
    let mu = discrete_expectation(pmf, x)
    discrete_expectation(pmf, cx) = c * mu
    discrete_expectation(pmf, cx) * discrete_expectation(pmf, cx) = (c * mu) * (c * mu)
    (c * mu) * (c * mu) = c * (mu * (c * mu))
    mu * (c * mu) = c * (mu * mu)
    (c * mu) * (c * mu) = c * (c * (mu * mu))
    c * (c * (mu * mu)) = (c * c) * (mu * mu)
    discrete_expectation(pmf, cx) * discrete_expectation(pmf, cx) = (c * c) * (mu * mu)

    discrete_variance_formula[T](pmf, cx)
    discrete_variance(pmf, cx) = discrete_expectation(pmf, square_fn(cx)) - discrete_expectation(pmf, cx) * discrete_expectation(pmf, cx)
    discrete_variance(pmf, cx) = (c * c) * discrete_expectation(pmf, square_fn(x)) - (c * c) * (mu * mu)
    (c * c) * discrete_expectation(pmf, square_fn(x)) - (c * c) * (mu * mu) = (c * c) * (discrete_expectation(pmf, square_fn(x)) - mu * mu)
    discrete_variance_formula[T](pmf, x)
    discrete_variance(pmf, x) = discrete_expectation(pmf, square_fn(x)) - mu * mu
    discrete_variance(pmf, cx) = (c * c) * discrete_variance(pmf, x)
}

/// Expectation is additive in a real constant: `E(X + c) = E(X) + c`.
theorem discrete_expectation_add_const[T](pmf: DiscretePMF[T], x: T -> Real, c: Real) {
    discrete_expectation(pmf, add_fn(x, const_real_fn[T](c)))
        = discrete_expectation(pmf, x) + c
} by {
    discrete_expectation_add[T](pmf, x, const_real_fn[T](c))
    discrete_expectation_const[T](pmf, c)
    discrete_expectation(pmf, const_real_fn[T](c)) = c
}

/// Expectation subtracts a real constant: `E(X + (-c)) = E(X) - c`.
theorem discrete_expectation_sub_const[T](pmf: DiscretePMF[T], x: T -> Real, c: Real) {
    discrete_expectation(pmf, add_fn(x, const_real_fn[T](-c)))
        = discrete_expectation(pmf, x) - c
} by {
    discrete_expectation_add_const[T](pmf, x, -c)
    discrete_expectation(pmf, x) + -c = discrete_expectation(pmf, x) - c
}

/// Variance is invariant under adding a real constant: `Var(X + c) = Var(X)`.
theorem discrete_variance_add_const[T](pmf: DiscretePMF[T], x: T -> Real, c: Real) {
    discrete_variance(pmf, add_fn(x, const_real_fn[T](c))) = discrete_variance(pmf, x)
} by {
    let y = add_fn(x, const_real_fn[T](c))
    let mu_x = discrete_expectation(pmf, x)
    discrete_expectation_add_const[T](pmf, x, c)
    let mu_y = discrete_expectation(pmf, y)
    mu_y = mu_x + c
    forall(t: T) {
        y(t) = add_fn(x, const_real_fn[T](c), t)
        add_fn(x, const_real_fn[T](c), t) = x(t) + const_real_fn[T](c, t)
        const_real_fn_apply[T](c, t)
        const_real_fn[T](c, t) = c
        y(t) = x(t) + c
        y(t) - mu_y = (x(t) + c) - (mu_x + c)
        (x(t) + c) - (mu_x + c) = x(t) + c + -(mu_x + c)
        -(mu_x + c) = -mu_x + -c
        x(t) + c + -(mu_x + c) = x(t) + c + (-mu_x + -c)
        x(t) + c + (-mu_x + -c) = x(t) + (c + (-mu_x + -c))
        c + (-mu_x + -c) = c + (-c + -mu_x)
        c + (-c + -mu_x) = (c + -c) + -mu_x
        c + -c = Real.0
        (c + -c) + -mu_x = Real.0 + -mu_x
        Real.0 + -mu_x = -mu_x
        c + (-mu_x + -c) = -mu_x
        x(t) + (c + (-mu_x + -c)) = x(t) + -mu_x
        x(t) + -mu_x = x(t) - mu_x
        y(t) - mu_y = x(t) - mu_x
        square_dev(y, mu_y, t) = (y(t) - mu_y) * (y(t) - mu_y)
        square_dev(x, mu_x, t) = (x(t) - mu_x) * (x(t) - mu_x)
        (y(t) - mu_y) * (y(t) - mu_y) = (x(t) - mu_x) * (x(t) - mu_x)
        square_dev(y, mu_y, t) = square_dev(x, mu_x, t)
    }
    function_extensionality[T, Real](square_dev(y, mu_y), square_dev(x, mu_x))
    square_dev(y, mu_y) = square_dev(x, mu_x)
    discrete_variance(pmf, y) = discrete_expectation(pmf, square_dev(y, mu_y))
    discrete_variance(pmf, x) = discrete_expectation(pmf, square_dev(x, mu_x))
}

/// Variance is invariant under subtracting a real constant: `Var(X + (-c)) = Var(X)`.
theorem discrete_variance_sub_const[T](pmf: DiscretePMF[T], x: T -> Real, c: Real) {
    discrete_variance(pmf, add_fn(x, const_real_fn[T](-c))) = discrete_variance(pmf, x)
} by {
    discrete_variance_add_const[T](pmf, x, -c)
}
