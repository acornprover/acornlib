/// The hypergeometric distribution: the law of the number of successes in a
/// sample of `n` items drawn without replacement from a population of `big_n`
/// items of which `big_k` are successes.

from nat import Nat, from_nat, from_nat_zero, from_nat_one, from_nat_add, from_nat_mul,
    alt_induction, add_sub, lt_or_lte, lt_suc, lt_and_lte
from real import Real, mul_nonneg, mul_div_cancel, from_nat_real_pos_of_ne_zero
from finite_set import FiniteSet, finite_set_sum
from algebra.semigroup import mul_fn
from combinatorics import binom, binom_pos, choose_zero, choose_out_of_bounds, vandermonde, vandermonde_term
from data.nat.nat_range_set import range_set, range_set_contains_eq
from data.nat.nat_range_sum import range_sum, range_sum_zero, range_sum_suc
from data.nat.nat_range_sum_bridge import range_sum_eq_partial, range_sum_eq_finite_set_sum
from list import partial, partial_pointwise_eq, partial_scalar_mul, partial_zero
from data.basic.functions import compose, function_extensionality
from probability.binomial import from_nat_nonneg_real, gte_zero_unfold
from probability.discrete_pmf import DiscretePMF, is_pmf
from ordered_field import inverse_of_nonnegative_is_nonnegative

numerals Nat

/// The hypergeometric probability mass at `k`:
/// P(X = k) = C(big_k,k) C(big_n-big_k,n-k) / C(big_n,n) for k = 0, ..., n,
/// and P(X = k) = 0 for k > n (a sample of size `n` cannot contain more than
/// `n` successes).
define hypergeometric_mass(big_n: Nat, big_k: Nat, n: Nat, k: Nat) -> Real {
    if k <= n {
        from_nat[Real](big_k.binom(k)) * from_nat[Real]((big_n - big_k).binom(n - k)) / from_nat[Real](big_n.binom(n))
    } else {
        Real.0
    }
}

/// The finite support `{0, ..., n}` of the hypergeometric distribution.
define hypergeometric_support(n: Nat) -> FiniteSet[Nat] {
    range_set(n.suc)
}

/// The natural-valued numerator C(big_k,k) C(big_n-big_k,n-k) of the hypergeometric mass.
define hypergeometric_nat_num(big_n: Nat, big_k: Nat, n: Nat, k: Nat) -> Nat {
    big_k.binom(k) * (big_n - big_k).binom(n - k)
}

/// The real-valued numerator of the hypergeometric mass.
define hypergeometric_num_term(big_n: Nat, big_k: Nat, n: Nat, k: Nat) -> Real {
    from_nat[Real](big_k.binom(k)) * from_nat[Real]((big_n - big_k).binom(n - k))
}

/// The real value of the natural-valued numerator is the real-valued numerator.
theorem hypergeometric_num_term_eq_compose(big_n: Nat, big_k: Nat, n: Nat, k: Nat) {
    hypergeometric_num_term(big_n, big_k, n, k) =
        compose(from_nat[Real], hypergeometric_nat_num(big_n, big_k, n), k)
} by {
    hypergeometric_num_term(big_n, big_k, n, k) =
        from_nat[Real](big_k.binom(k)) * from_nat[Real]((big_n - big_k).binom(n - k))
    from_nat_mul[Real](big_k.binom(k), (big_n - big_k).binom(n - k))
    from_nat[Real](big_k.binom(k) * (big_n - big_k).binom(n - k)) =
        from_nat[Real](big_k.binom(k)) * from_nat[Real]((big_n - big_k).binom(n - k))
    hypergeometric_nat_num(big_n, big_k, n, k) = big_k.binom(k) * (big_n - big_k).binom(n - k)
    compose(from_nat[Real], hypergeometric_nat_num(big_n, big_k, n), k) =
        from_nat[Real](hypergeometric_nat_num(big_n, big_k, n, k))
    hypergeometric_num_term(big_n, big_k, n, k) =
        compose(from_nat[Real], hypergeometric_nat_num(big_n, big_k, n), k)
}

/// The natural-valued numerator is the Vandermonde convolution term.
theorem hypergeometric_nat_num_eq_vandermonde_term(big_n: Nat, big_k: Nat, n: Nat, k: Nat) {
    hypergeometric_nat_num(big_n, big_k, n, k) = vandermonde_term(big_k, big_n - big_k, n, k)
} by {
    hypergeometric_nat_num(big_n, big_k, n, k) = big_k.binom(k) * (big_n - big_k).binom(n - k)
    vandermonde_term(big_k, big_n - big_k, n, k) = big_k.binom(k) * (big_n - big_k).binom(n - k)
    hypergeometric_nat_num(big_n, big_k, n, k) = vandermonde_term(big_k, big_n - big_k, n, k)
}

/// `from_nat` commutes with range sums: the real value of a natural range sum
/// is the range sum of the real values.
lemma from_nat_range_sum(f: Nat -> Nat, n: Nat) {
    from_nat[Real](range_sum(f, n)) = range_sum(compose(from_nat[Real], f), n)
} by {
    define p(x: Nat) -> Bool {
        from_nat[Real](range_sum(f, x)) = range_sum(compose(from_nat[Real], f), x)
    }
    range_sum_zero(f)
    range_sum(f, Nat.0) = Nat.0
    from_nat_zero[Real]
    from_nat[Real](Nat.0) = Real.0
    range_sum_zero(compose(from_nat[Real], f))
    range_sum(compose(from_nat[Real], f), Nat.0) = Real.0
    p(Nat.0)
    forall(k: Nat) {
        if p(k) {
            range_sum_suc(f, k)
            range_sum(f, k.suc) = range_sum(f, k) + f(k)
            from_nat_add[Real](range_sum(f, k), f(k))
            from_nat[Real](range_sum(f, k) + f(k)) =
                from_nat[Real](range_sum(f, k)) + from_nat[Real](f(k))
            from_nat[Real](range_sum(f, k)) = range_sum(compose(from_nat[Real], f), k)
            from_nat[Real](range_sum(f, k.suc)) =
                range_sum(compose(from_nat[Real], f), k) + from_nat[Real](f(k))
            compose(from_nat[Real], f, k) = from_nat[Real](f(k))
            range_sum_suc(compose(from_nat[Real], f), k)
            range_sum(compose(from_nat[Real], f), k.suc) =
                range_sum(compose(from_nat[Real], f), k) + compose(from_nat[Real], f, k)
            from_nat[Real](range_sum(f, k.suc)) =
                range_sum(compose(from_nat[Real], f), k.suc)
            p(k.suc)
        }
    }
    p(Nat.0) and forall(k: Nat) { p(k) implies p(k.suc) }
    alt_induction(p)
    forall(x: Nat) { p(x) }
    p(n)
    p(n) = (from_nat[Real](range_sum(f, n)) = range_sum(compose(from_nat[Real], f), n))
    from_nat[Real](range_sum(f, n)) = range_sum(compose(from_nat[Real], f), n)
}

/// On the support, the mass function is the scalar multiple of the real-valued
/// numerator by the reciprocal of C(big_n,n).
lemma hypergeometric_mass_eq_scalar_num_of_le(big_n: Nat, big_k: Nat, n: Nat, k: Nat) {
    k <= n implies
        hypergeometric_mass(big_n, big_k, n, k) =
            mul_fn(Real.1 / from_nat[Real](big_n.binom(n)), hypergeometric_num_term(big_n, big_k, n), k)
} by {
    if k <= n {
        hypergeometric_mass(big_n, big_k, n, k) =
            from_nat[Real](big_k.binom(k)) * from_nat[Real]((big_n - big_k).binom(n - k)) / from_nat[Real](big_n.binom(n))
        from_nat[Real](big_k.binom(k)) * from_nat[Real]((big_n - big_k).binom(n - k)) / from_nat[Real](big_n.binom(n)) =
            (from_nat[Real](big_k.binom(k)) * from_nat[Real]((big_n - big_k).binom(n - k))) * (Real.1 / from_nat[Real](big_n.binom(n)))
        hypergeometric_num_term(big_n, big_k, n, k) =
            from_nat[Real](big_k.binom(k)) * from_nat[Real]((big_n - big_k).binom(n - k))
        (from_nat[Real](big_k.binom(k)) * from_nat[Real]((big_n - big_k).binom(n - k))) * (Real.1 / from_nat[Real](big_n.binom(n))) =
            (Real.1 / from_nat[Real](big_n.binom(n))) * (from_nat[Real](big_k.binom(k)) * from_nat[Real]((big_n - big_k).binom(n - k)))
        mul_fn(Real.1 / from_nat[Real](big_n.binom(n)), hypergeometric_num_term(big_n, big_k, n), k) =
            (Real.1 / from_nat[Real](big_n.binom(n))) * hypergeometric_num_term(big_n, big_k, n, k)
        hypergeometric_mass(big_n, big_k, n, k) =
            mul_fn(Real.1 / from_nat[Real](big_n.binom(n)), hypergeometric_num_term(big_n, big_k, n), k)
    }
}

/// The hypergeometric probabilities sum to one: Σ_{k=0}^{n} C(big_k,k) C(big_n-big_k,n-k)
/// / C(big_n,n) = 1, by Vandermonde's identity.
theorem hypergeometric_mass_total(big_n: Nat, big_k: Nat, n: Nat) {
    big_k <= big_n and n <= big_n implies
        finite_set_sum(hypergeometric_support(n), hypergeometric_mass(big_n, big_k, n)) = Real.1
} by {
    if big_k <= big_n and n <= big_n {
        big_k <= big_n
        n <= big_n
        range_sum_eq_finite_set_sum(hypergeometric_mass(big_n, big_k, n), n.suc)
        finite_set_sum(hypergeometric_support(n), hypergeometric_mass(big_n, big_k, n)) =
            range_sum(hypergeometric_mass(big_n, big_k, n), n.suc)
        range_sum_eq_partial(hypergeometric_mass(big_n, big_k, n), n.suc)
        range_sum(hypergeometric_mass(big_n, big_k, n), n.suc) =
            partial(hypergeometric_mass(big_n, big_k, n), n.suc)
        finite_set_sum(hypergeometric_support(n), hypergeometric_mass(big_n, big_k, n)) =
            partial(hypergeometric_mass(big_n, big_k, n), n.suc)

        forall(k: Nat) {
            if k < n.suc {
                hypergeometric_mass_eq_scalar_num_of_le(big_n, big_k, n, k)
                hypergeometric_mass(big_n, big_k, n, k) =
                    mul_fn(Real.1 / from_nat[Real](big_n.binom(n)), hypergeometric_num_term(big_n, big_k, n), k)
            }
        }
        partial_pointwise_eq(hypergeometric_mass(big_n, big_k, n),
            mul_fn(Real.1 / from_nat[Real](big_n.binom(n)), hypergeometric_num_term(big_n, big_k, n)), n.suc)
        partial(hypergeometric_mass(big_n, big_k, n), n.suc) =
            partial(mul_fn(Real.1 / from_nat[Real](big_n.binom(n)), hypergeometric_num_term(big_n, big_k, n)), n.suc)
        partial_scalar_mul(Real.1 / from_nat[Real](big_n.binom(n)), hypergeometric_num_term(big_n, big_k, n), n.suc)
        (Real.1 / from_nat[Real](big_n.binom(n))) * partial(hypergeometric_num_term(big_n, big_k, n), n.suc) =
            partial(mul_fn(Real.1 / from_nat[Real](big_n.binom(n)), hypergeometric_num_term(big_n, big_k, n)), n.suc)
        partial(hypergeometric_mass(big_n, big_k, n), n.suc) =
            (Real.1 / from_nat[Real](big_n.binom(n))) * partial(hypergeometric_num_term(big_n, big_k, n), n.suc)

        forall(k: Nat) {
            if k < n.suc {
                hypergeometric_num_term_eq_compose(big_n, big_k, n, k)
                hypergeometric_num_term(big_n, big_k, n, k) =
                    compose(from_nat[Real], hypergeometric_nat_num(big_n, big_k, n), k)
            }
        }
        partial_pointwise_eq(hypergeometric_num_term(big_n, big_k, n),
            compose(from_nat[Real], hypergeometric_nat_num(big_n, big_k, n)), n.suc)
        partial(hypergeometric_num_term(big_n, big_k, n), n.suc) =
            partial(compose(from_nat[Real], hypergeometric_nat_num(big_n, big_k, n)), n.suc)
        range_sum_eq_partial(compose(from_nat[Real], hypergeometric_nat_num(big_n, big_k, n)), n.suc)
        range_sum(compose(from_nat[Real], hypergeometric_nat_num(big_n, big_k, n)), n.suc) =
            partial(compose(from_nat[Real], hypergeometric_nat_num(big_n, big_k, n)), n.suc)
        from_nat_range_sum(hypergeometric_nat_num(big_n, big_k, n), n.suc)
        from_nat[Real](range_sum(hypergeometric_nat_num(big_n, big_k, n), n.suc)) =
            range_sum(compose(from_nat[Real], hypergeometric_nat_num(big_n, big_k, n)), n.suc)
        partial(hypergeometric_num_term(big_n, big_k, n), n.suc) =
            from_nat[Real](range_sum(hypergeometric_nat_num(big_n, big_k, n), n.suc))

        forall(k: Nat) {
            if k < n.suc {
                hypergeometric_nat_num_eq_vandermonde_term(big_n, big_k, n, k)
                hypergeometric_nat_num(big_n, big_k, n, k) = vandermonde_term(big_k, big_n - big_k, n, k)
            }
        }
        partial_pointwise_eq(hypergeometric_nat_num(big_n, big_k, n), vandermonde_term(big_k, big_n - big_k, n), n.suc)
        partial(hypergeometric_nat_num(big_n, big_k, n), n.suc) =
            partial(vandermonde_term(big_k, big_n - big_k, n), n.suc)
        range_sum_eq_partial(hypergeometric_nat_num(big_n, big_k, n), n.suc)
        range_sum(hypergeometric_nat_num(big_n, big_k, n), n.suc) =
            partial(hypergeometric_nat_num(big_n, big_k, n), n.suc)
        range_sum_eq_partial(vandermonde_term(big_k, big_n - big_k, n), n.suc)
        range_sum(vandermonde_term(big_k, big_n - big_k, n), n.suc) =
            partial(vandermonde_term(big_k, big_n - big_k, n), n.suc)
        range_sum(hypergeometric_nat_num(big_n, big_k, n), n.suc) =
            range_sum(vandermonde_term(big_k, big_n - big_k, n), n.suc)
        vandermonde(big_k, big_n - big_k, n)
        partial(vandermonde_term(big_k, big_n - big_k, n), n.suc) = (big_k + (big_n - big_k)).binom(n)
        range_sum(vandermonde_term(big_k, big_n - big_k, n), n.suc) = (big_k + (big_n - big_k)).binom(n)
        add_sub(big_n, big_k)
        big_n - big_k + big_k = big_n
        big_k + (big_n - big_k) = big_n
        (big_k + (big_n - big_k)).binom(n) = big_n.binom(n)
        range_sum(hypergeometric_nat_num(big_n, big_k, n), n.suc) = big_n.binom(n)

        partial(hypergeometric_num_term(big_n, big_k, n), n.suc) = from_nat[Real](big_n.binom(n))
        partial(hypergeometric_mass(big_n, big_k, n), n.suc) =
            (Real.1 / from_nat[Real](big_n.binom(n))) * from_nat[Real](big_n.binom(n))
        binom_pos(big_n, n)
        Nat.0 < big_n.binom(n)
        big_n.binom(n) != Nat.0
        from_nat_real_pos_of_ne_zero(big_n.binom(n))
        from_nat[Real](big_n.binom(n)) > Real.0
        from_nat[Real](big_n.binom(n)) != Real.0
        mul_div_cancel(Real.1, from_nat[Real](big_n.binom(n)))
        from_nat[Real](big_n.binom(n)) * (Real.1 / from_nat[Real](big_n.binom(n))) = Real.1
        (Real.1 / from_nat[Real](big_n.binom(n))) * from_nat[Real](big_n.binom(n)) = Real.1
        partial(hypergeometric_mass(big_n, big_k, n), n.suc) = Real.1
        finite_set_sum(hypergeometric_support(n), hypergeometric_mass(big_n, big_k, n)) = Real.1
    }
}

/// The hypergeometric mass vanishes above the sample size `n`.
theorem hypergeometric_mass_zero_above(big_n: Nat, big_k: Nat, n: Nat, k: Nat) {
    not k <= n implies hypergeometric_mass(big_n, big_k, n, k) = Real.0
} by {
    if not k <= n {
        if k <= n {
            false
        } else {
            hypergeometric_mass(big_n, big_k, n, k) = Real.0
        }
        hypergeometric_mass(big_n, big_k, n, k) = Real.0
    }
}

/// The hypergeometric mass vanishes outside the support {0, ..., n}.
theorem hypergeometric_mass_zero_outside(big_n: Nat, big_k: Nat, n: Nat, k: Nat) {
    not hypergeometric_support(n).contains(k) implies
        hypergeometric_mass(big_n, big_k, n, k) = Real.0
} by {
    if not hypergeometric_support(n).contains(k) {
        hypergeometric_support(n) = range_set(n.suc)
        hypergeometric_support(n).contains(k) = range_set(n.suc).contains(k)
        range_set_contains_eq(n.suc, k)
        range_set(n.suc).contains(k) = (k < n.suc)
        hypergeometric_support(n).contains(k) = (k < n.suc)
        not (k < n.suc)
        lt_or_lte(k, n.suc)
        k < n.suc or n.suc <= k
        n.suc <= k
        lt_suc(n)
        n < n.suc
        lt_and_lte(n, n.suc, k)
        n < k
        not (k <= n)
        hypergeometric_mass_zero_above(big_n, big_k, n, k)
        hypergeometric_mass(big_n, big_k, n, k) = Real.0
    }
}

/// The hypergeometric mass is nonnegative when `big_k <= big_n` and `n <= big_n`.
theorem hypergeometric_mass_nonneg(big_n: Nat, big_k: Nat, n: Nat, k: Nat) {
    big_k <= big_n and n <= big_n implies Real.0 <= hypergeometric_mass(big_n, big_k, n, k)
} by {
    if big_k <= big_n and n <= big_n {
        n <= big_n
        if k <= n {
            hypergeometric_mass(big_n, big_k, n, k) =
                from_nat[Real](big_k.binom(k)) * from_nat[Real]((big_n - big_k).binom(n - k)) / from_nat[Real](big_n.binom(n))
            from_nat_nonneg_real(big_k.binom(k))
            Real.0 <= from_nat[Real](big_k.binom(k))
            from_nat_nonneg_real((big_n - big_k).binom(n - k))
            Real.0 <= from_nat[Real]((big_n - big_k).binom(n - k))
            gte_zero_unfold(from_nat[Real](big_k.binom(k)))
            from_nat[Real](big_k.binom(k)) >= Real.0
            gte_zero_unfold(from_nat[Real]((big_n - big_k).binom(n - k)))
            from_nat[Real]((big_n - big_k).binom(n - k)) >= Real.0
            mul_nonneg(from_nat[Real](big_k.binom(k)), from_nat[Real]((big_n - big_k).binom(n - k)))
            from_nat[Real](big_k.binom(k)) * from_nat[Real]((big_n - big_k).binom(n - k)) >= Real.0
            gte_zero_unfold(from_nat[Real](big_k.binom(k)) * from_nat[Real]((big_n - big_k).binom(n - k)))
            Real.0 <= from_nat[Real](big_k.binom(k)) * from_nat[Real]((big_n - big_k).binom(n - k))
            binom_pos(big_n, n)
            Nat.0 < big_n.binom(n)
            big_n.binom(n) != Nat.0
            from_nat_real_pos_of_ne_zero(big_n.binom(n))
            from_nat[Real](big_n.binom(n)) > Real.0
            from_nat[Real](big_n.binom(n)) >= Real.0
            inverse_of_nonnegative_is_nonnegative[Real](from_nat[Real](big_n.binom(n)))
            Real.0 <= from_nat[Real](big_n.binom(n)).inverse
            Real.1 / from_nat[Real](big_n.binom(n)) = from_nat[Real](big_n.binom(n)).inverse
            Real.0 <= Real.1 / from_nat[Real](big_n.binom(n))
            gte_zero_unfold(Real.1 / from_nat[Real](big_n.binom(n)))
            Real.1 / from_nat[Real](big_n.binom(n)) >= Real.0
            mul_nonneg(from_nat[Real](big_k.binom(k)) * from_nat[Real]((big_n - big_k).binom(n - k)),
                Real.1 / from_nat[Real](big_n.binom(n)))
            (from_nat[Real](big_k.binom(k)) * from_nat[Real]((big_n - big_k).binom(n - k))) * (Real.1 / from_nat[Real](big_n.binom(n))) >= Real.0
            gte_zero_unfold((from_nat[Real](big_k.binom(k)) * from_nat[Real]((big_n - big_k).binom(n - k))) * (Real.1 / from_nat[Real](big_n.binom(n))))
            Real.0 <= (from_nat[Real](big_k.binom(k)) * from_nat[Real]((big_n - big_k).binom(n - k))) * (Real.1 / from_nat[Real](big_n.binom(n)))
            from_nat[Real](big_k.binom(k)) * from_nat[Real]((big_n - big_k).binom(n - k)) / from_nat[Real](big_n.binom(n)) =
                (from_nat[Real](big_k.binom(k)) * from_nat[Real]((big_n - big_k).binom(n - k))) * (Real.1 / from_nat[Real](big_n.binom(n)))
            Real.0 <= from_nat[Real](big_k.binom(k)) * from_nat[Real]((big_n - big_k).binom(n - k)) / from_nat[Real](big_n.binom(n))
            Real.0 <= hypergeometric_mass(big_n, big_k, n, k)
        }
        if not k <= n {
            hypergeometric_mass_zero_above(big_n, big_k, n, k)
            hypergeometric_mass(big_n, big_k, n, k) = Real.0
            Real.0 <= Real.0
            Real.0 <= hypergeometric_mass(big_n, big_k, n, k)
        }
        k <= n or not k <= n
        Real.0 <= hypergeometric_mass(big_n, big_k, n, k)
    }
}

/// For `big_k <= big_n` and `n <= big_n`, the hypergeometric mass data is a
/// valid probability mass function.
theorem hypergeometric_is_pmf(big_n: Nat, big_k: Nat, n: Nat) {
    big_k <= big_n and n <= big_n implies is_pmf(hypergeometric_support(n), hypergeometric_mass(big_n, big_k, n))
} by {
    if big_k <= big_n and n <= big_n {
        let cond_nonneg: Bool = forall(x: Nat) { Real.0 <= hypergeometric_mass(big_n, big_k, n, x) }
        let cond_zero: Bool = forall(x: Nat) {
            not hypergeometric_support(n).contains(x) implies hypergeometric_mass(big_n, big_k, n, x) = Real.0
        }
        let cond_sum: Bool = finite_set_sum(hypergeometric_support(n), hypergeometric_mass(big_n, big_k, n)) = Real.1
        forall(x: Nat) {
            hypergeometric_mass_nonneg(big_n, big_k, n, x)
            Real.0 <= hypergeometric_mass(big_n, big_k, n, x)
        }
        cond_nonneg
        forall(x: Nat) {
            hypergeometric_mass_zero_outside(big_n, big_k, n, x)
        }
        cond_zero
        hypergeometric_mass_total(big_n, big_k, n)
        cond_sum
        cond_nonneg and cond_zero
        cond_nonneg and cond_zero and cond_sum
        is_pmf(hypergeometric_support(n), hypergeometric_mass(big_n, big_k, n)) =
            (cond_nonneg and cond_zero and cond_sum)
        is_pmf(hypergeometric_support(n), hypergeometric_mass(big_n, big_k, n))
    }
}

/// A bundled hypergeometric PMF exists for every valid parameter triple.
theorem hypergeometric_pmf_exists(big_n: Nat, big_k: Nat, n: Nat) {
    big_k <= big_n and n <= big_n implies exists(pmf: DiscretePMF[Nat]) {
        DiscretePMF[Nat].new(hypergeometric_support(n), hypergeometric_mass(big_n, big_k, n)) = Option.some(pmf)
    }
} by {
    if big_k <= big_n and n <= big_n {
        hypergeometric_is_pmf(big_n, big_k, n)
        is_pmf(hypergeometric_support(n), hypergeometric_mass(big_n, big_k, n))
        exists(pmf: DiscretePMF[Nat]) {
            DiscretePMF[Nat].new(hypergeometric_support(n), hypergeometric_mass(big_n, big_k, n)) = Option.some(pmf)
        }
    }
}

/// The probability of drawing no successes is C(big_n - big_k, n) / C(big_n, n).
theorem hypergeometric_mass_zero_successes(big_n: Nat, big_k: Nat, n: Nat) {
    hypergeometric_mass(big_n, big_k, n, Nat.0) =
        from_nat[Real]((big_n - big_k).binom(n)) / from_nat[Real](big_n.binom(n))
} by {
    if Nat.0 <= n {
        hypergeometric_mass(big_n, big_k, n, Nat.0) =
            from_nat[Real](big_k.binom(0)) * from_nat[Real]((big_n - big_k).binom(n - 0)) / from_nat[Real](big_n.binom(n))
        choose_zero(big_k)
        big_k.binom(0) = Nat.1
        from_nat_one[Real]
        from_nat[Real](Nat.1) = Real.1
        from_nat[Real](big_k.binom(0)) = Real.1
        n - 0 = n
        (big_n - big_k).binom(n - 0) = (big_n - big_k).binom(n)
        Real.1 * from_nat[Real]((big_n - big_k).binom(n)) / from_nat[Real](big_n.binom(n)) =
            from_nat[Real]((big_n - big_k).binom(n)) / from_nat[Real](big_n.binom(n))
        hypergeometric_mass(big_n, big_k, n, Nat.0) =
            from_nat[Real]((big_n - big_k).binom(n)) / from_nat[Real](big_n.binom(n))
    }
    Nat.0 <= n
}
