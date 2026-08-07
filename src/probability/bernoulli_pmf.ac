from real import Real
from finite_set import FiniteSet, fs_empty, fs_insert, finite_set_sum, finite_set_sum_empty,
    finite_set_sum_insert
from data.basic.set import Set, empty_set_contains_eq, insert_contains_eq

/// The Bernoulli mass function with parameter `p`:
/// `p` at `true` and `1 - p` at `false`.
define bernoulli_mass(p: Real, b: Bool) -> Real {
    if b {
        p
    } else {
        Real.1 - p
    }
}

/// The Bernoulli mass at `true` is `p`.
theorem bernoulli_mass_true(p: Real) {
    bernoulli_mass(p, true) = p
}

/// The Bernoulli mass at `false` is `1 - p`.
theorem bernoulli_mass_false(p: Real) {
    bernoulli_mass(p, false) = Real.1 - p
}

/// The two-element Boolean support `{false, true}` of any Bernoulli distribution.
let bernoulli_support: FiniteSet[Bool] =
    fs_insert(fs_insert(fs_empty[Bool](Set[Bool].empty_set), false), true)

/// The empty Boolean finite set does not contain `false`.
theorem fs_empty_bool_no_false {
    not fs_empty[Bool](Set[Bool].empty_set).contains(false)
} by {
    fs_empty[Bool](Set[Bool].empty_set).underlying_set = Set[Bool].empty_set
    empty_set_contains_eq[Bool](false)
}

/// After inserting only `false` into the empty set, `true` is not yet a member.
theorem fs_insert_false_no_true {
    not fs_insert(fs_empty[Bool](Set[Bool].empty_set), false).contains(true)
} by {
    let e0: FiniteSet[Bool] = fs_empty[Bool](Set[Bool].empty_set)
    e0.underlying_set = Set[Bool].empty_set
    fs_insert(e0, false).underlying_set = e0.underlying_set.insert(false)
    insert_contains_eq(Set[Bool].empty_set, false, true)
    empty_set_contains_eq[Bool](true)
    true != false
}

/// The Bernoulli support contains every Boolean value.
theorem bernoulli_support_contains_all(b: Bool) {
    bernoulli_support.contains(b)
} by {
    let e0: FiniteSet[Bool] = fs_empty[Bool](Set[Bool].empty_set)
    let s1: FiniteSet[Bool] = fs_insert(e0, false)
    e0.underlying_set = Set[Bool].empty_set
    s1.underlying_set = e0.underlying_set.insert(false)
    insert_contains_eq(Set[Bool].empty_set, false, b)
    s1.underlying_set.contains(b) = (b = false or Set[Bool].empty_set.contains(b))
    empty_set_contains_eq[Bool](b)

    fs_insert(s1, true).underlying_set = s1.underlying_set.insert(true)
    bernoulli_support.underlying_set = s1.underlying_set.insert(true)
    insert_contains_eq(s1.underlying_set, true, b)
    bernoulli_support.underlying_set.contains(b) = (b = true or s1.underlying_set.contains(b))
    if b {
        bernoulli_support.underlying_set.contains(b)
    } else {
        bernoulli_support.underlying_set.contains(b)
    }
}

/// The total mass of the Bernoulli function over its support is `1`.
theorem bernoulli_support_sum(p: Real) {
    finite_set_sum(bernoulli_support, bernoulli_mass(p)) = Real.1
} by {
    let e0: FiniteSet[Bool] = fs_empty[Bool](Set[Bool].empty_set)
    let s1: FiniteSet[Bool] = fs_insert(e0, false)
    let f: Bool -> Real = bernoulli_mass(p)

    finite_set_sum_empty[Bool, Real](f)

    fs_empty_bool_no_false
    finite_set_sum_insert[Bool, Real](e0, false, f)
    finite_set_sum(s1, f) = f(false) + finite_set_sum(e0, f)
    finite_set_sum(s1, f) = (Real.1 - p) + Real.0

    fs_insert_false_no_true
    finite_set_sum_insert[Bool, Real](s1, true, f)
    finite_set_sum(fs_insert(s1, true), f) = f(true) + finite_set_sum(s1, f)
    bernoulli_support = fs_insert(s1, true)
    bernoulli_mass(p, true) = p
    finite_set_sum(bernoulli_support, f) = p + (Real.1 - p)
    (Real.1 + p) + -p = Real.1 + (p + -p)
    p + (Real.1 - p) = Real.1
}

/// Subtraction of `p` from `Real.1` is nonnegative when `p <= Real.1`.
theorem one_sub_nonneg(p: Real) {
    p <= Real.1 implies Real.0 <= Real.1 - p
} by {
    if p <= Real.1 {
        p + -p <= Real.1 + -p
        Real.0 <= Real.1 - p
    }
}

/// The Bernoulli mass function is nonnegative when `0 <= p <= 1`.
theorem bernoulli_mass_nonneg(p: Real, b: Bool) {
    Real.0 <= p and p <= Real.1 implies Real.0 <= bernoulli_mass(p, b)
} by {
    if Real.0 <= p and p <= Real.1 {
        if b {
            Real.0 <= bernoulli_mass(p, b)
        } else {
            bernoulli_mass(p, b) = Real.1 - p
            one_sub_nonneg(p)
            Real.0 <= bernoulli_mass(p, b)
        }
    }
}

from probability.discrete_pmf import DiscretePMF, is_pmf

/// If a finite set contains every element of `T`, the vanishing-outside-support
/// clause of `is_pmf` is vacuously satisfied for any real-valued `mass`.
theorem cond_zero_from_full[T](support: FiniteSet[T], mass: T -> Real) {
    (forall(x: T) { support.contains(x) }) implies
        (forall(y: T) { not support.contains(y) implies mass(y) = Real.0 })
}

/// For `0 <= p <= 1`, `(bernoulli_support, bernoulli_mass(p))` is a valid PMF.
theorem bernoulli_is_pmf(p: Real) {
    Real.0 <= p and p <= Real.1 implies is_pmf(bernoulli_support, bernoulli_mass(p))
} by {
    if Real.0 <= p and p <= Real.1 {
        let cond_nonneg: Bool = forall(x: Bool) { Real.0 <= bernoulli_mass(p, x) }
        let cond_zero: Bool = forall(x: Bool) {
            not bernoulli_support.contains(x) implies bernoulli_mass(p, x) = Real.0
        }
        forall(x: Bool) {
            bernoulli_mass_nonneg(p, x)
            Real.0 <= bernoulli_mass(p, x)
        }
        forall(x: Bool) {
            bernoulli_support_contains_all(x)
        }
        cond_zero_from_full[Bool](bernoulli_support, bernoulli_mass(p))
        cond_zero
        bernoulli_support_sum(p)
        let cond_sum: Bool = finite_set_sum(bernoulli_support, bernoulli_mass(p)) = Real.1
        is_pmf(bernoulli_support, bernoulli_mass(p)) = (cond_nonneg and cond_zero and cond_sum)
        is_pmf(bernoulli_support, bernoulli_mass(p))
    }
}

/// A bundled Bernoulli PMF can be constructed for every parameter in `[0, 1]`.
theorem bernoulli_pmf_exists(p: Real) {
    Real.0 <= p and p <= Real.1 implies exists(pmf: DiscretePMF[Bool]) {
        DiscretePMF[Bool].new(bernoulli_support, bernoulli_mass(p)) = Option.some(pmf)
    }
} by {
    if Real.0 <= p and p <= Real.1 {
        bernoulli_is_pmf(p)
        is_pmf(bernoulli_support, bernoulli_mass(p))
        exists(pmf: DiscretePMF[Bool]) {
            DiscretePMF[Bool].new(bernoulli_support, bernoulli_mass(p)) = Option.some(pmf)
        }
    }
}

/// A bundled Bernoulli PMF constructed from the canonical data has Boolean support.
theorem bernoulli_pmf_new_support(p: Real, pmf: DiscretePMF[Bool]) {
    DiscretePMF[Bool].new(bernoulli_support, bernoulli_mass(p)) = Option.some(pmf) implies
        pmf.support = bernoulli_support
} by {
    if DiscretePMF[Bool].new(bernoulli_support, bernoulli_mass(p)) = Option.some(pmf) {
        pmf.support = bernoulli_support
    }
}
