from real import Real
from finite_set import FiniteSet, fs_insert, finite_set_empty_contains_eq,
    finite_set_sum, finite_set_sum_empty, finite_set_sum_insert
from data.basic.set import insert_contains_eq, insert_other_contains_imp_contains
from probability.discrete_pmf import DiscretePMF, is_pmf, discrete_pmf_mass_zero_outside,
    discrete_pmf_total_mass, pmf_event_prob,
    finite_event_indicator, finite_event_indicator_one_of_contains,
    finite_event_indicator_zero_of_not_contains, finite_event_indicator_nonneg,
    discrete_expectation, discrete_expectation_indicator_eq_event_prob,
    mass_weighted_value

/// The singleton support of the Dirac mass at `x`.
define dirac_support[T](x: T) -> FiniteSet[T] {
    fs_insert(FiniteSet.empty[T], x)
}

/// The Dirac mass at `x`, written as the indicator of the singleton support.
define dirac_mass[T](x: T, y: T) -> Real {
    finite_event_indicator(dirac_support(x), y)
}

/// The Dirac singleton support contains its distinguished point.
theorem dirac_support_contains_self[T](x: T) {
    dirac_support(x).contains(x)
} by {
    let e: FiniteSet[T] = FiniteSet.empty[T]
    fs_insert(e, x).underlying_set = e.underlying_set.insert(x)
    insert_contains_eq(e.underlying_set, x, x)
}

/// The Dirac singleton support omits every point distinct from its distinguished point.
theorem dirac_support_not_contains_of_not_eq[T](x: T, y: T) {
    x != y implies not dirac_support(x).contains(y)
} by {
    if x != y {
        if dirac_support(x).contains(y) {
            fs_insert(FiniteSet.empty[T], x).underlying_set = FiniteSet.empty[T].underlying_set.insert(x)
            insert_other_contains_imp_contains(FiniteSet.empty[T].underlying_set, y, x)
            FiniteSet.empty[T].underlying_set.contains(y)
            finite_set_empty_contains_eq[T](y)
            false
        }
        not dirac_support(x).contains(y)
    }
}

/// The Dirac mass at its distinguished point is one.
theorem dirac_mass_self[T](x: T) {
    dirac_mass(x, x) = Real.1
} by {
    dirac_support_contains_self(x)
    finite_event_indicator_one_of_contains(dirac_support(x), x)
}

/// The Dirac mass away from its distinguished point is zero.
theorem dirac_mass_zero_of_not_eq[T](x: T, y: T) {
    x != y implies dirac_mass(x, y) = Real.0
} by {
    if x != y {
        dirac_support_not_contains_of_not_eq(x, y)
        finite_event_indicator_zero_of_not_contains(dirac_support(x), y)
        dirac_mass(x, y) = Real.0
    }
}

/// The Dirac mass is nonnegative everywhere.
theorem dirac_mass_nonneg[T](x: T, y: T) {
    Real.0 <= dirac_mass(x, y)
} by {
    finite_event_indicator_nonneg(dirac_support(x), y)
}

/// The Dirac mass vanishes off its singleton support.
theorem dirac_mass_zero_outside_support[T](x: T, y: T) {
    not dirac_support(x).contains(y) implies dirac_mass(x, y) = Real.0
} by {
    finite_event_indicator_zero_of_not_contains(dirac_support(x), y)
}

/// The Dirac mass vanishes off its singleton support, uniformly in the point.
theorem dirac_mass_zero_outside_support_all[T](x: T) {
    forall(y: T) { not dirac_support(x).contains(y) implies dirac_mass(x, y) = Real.0 }
} by {
    forall(y: T) {
        dirac_mass_zero_outside_support(x, y)
    }
}

/// The Dirac mass has total mass one over its singleton support.
theorem dirac_support_sum[T](x: T) {
    finite_set_sum(dirac_support(x), dirac_mass(x)) = Real.1
} by {
    let f: T -> Real = dirac_mass(x)

    finite_set_sum_empty[T, Real](f)
    finite_set_sum(FiniteSet.empty[T], f) = Real.0

    finite_set_empty_contains_eq[T](x)
    finite_set_sum_insert[T, Real](FiniteSet.empty[T], x, f)
    finite_set_sum(fs_insert(FiniteSet.empty[T], x), f) = f(x) + finite_set_sum(FiniteSet.empty[T], f)

    dirac_mass_self(x)
}

/// The singleton Dirac mass data is a probability mass function.
theorem dirac_is_pmf[T](x: T) {
    is_pmf(dirac_support(x), dirac_mass(x))
} by {
    let cond_nonneg: Bool = forall(y: T) { Real.0 <= dirac_mass(x, y) }
    let cond_zero: Bool = forall(y: T) { not dirac_support(x).contains(y) implies dirac_mass(x, y) = Real.0 }
    let cond_sum: Bool = finite_set_sum(dirac_support(x), dirac_mass(x)) = Real.1

    forall(y: T) {
        dirac_mass_nonneg(x, y)
        Real.0 <= dirac_mass(x, y)
    }

    dirac_mass_zero_outside_support_all(x)
    cond_zero

    dirac_support_sum(x)

    is_pmf(dirac_support(x), dirac_mass(x)) = (cond_nonneg and cond_zero and cond_sum)
}

/// The bundled Dirac discrete PMF at `x`.
let dirac_pmf[T](x: T) -> result: DiscretePMF[T] satisfy {
    DiscretePMF.new(dirac_support(x), dirac_mass(x)) = Option.some(result)
} by {
    dirac_is_pmf(x)
}

/// The bundled Dirac PMF has the singleton support.
theorem dirac_pmf_support[T](x: T) {
    dirac_pmf(x).support = dirac_support(x)
} by {
    DiscretePMF.new(dirac_support(x), dirac_mass(x)) = Option.some(dirac_pmf(x))
}

/// The bundled Dirac PMF assigns mass one to its distinguished point.
theorem dirac_pmf_mass_self[T](x: T) {
    dirac_pmf(x).mass(x) = Real.1
} by {
    let pmf = dirac_pmf(x)
    dirac_pmf_support(x)
    discrete_pmf_total_mass[T](pmf)

    finite_set_sum_empty[T, Real](pmf.mass)
    finite_set_sum(FiniteSet.empty[T], pmf.mass) = Real.0
    finite_set_empty_contains_eq[T](x)
    finite_set_sum_insert[T, Real](FiniteSet.empty[T], x, pmf.mass)
    finite_set_sum(fs_insert(FiniteSet.empty[T], x), pmf.mass) =
        pmf.mass(x) + finite_set_sum(FiniteSet.empty[T], pmf.mass)
    finite_set_sum(dirac_support(x), pmf.mass) = pmf.mass(x)
}

/// The bundled Dirac PMF assigns zero mass away from its distinguished point.
theorem dirac_pmf_mass_zero_of_not_eq[T](x: T, y: T) {
    x != y implies dirac_pmf(x).mass(y) = Real.0
} by {
    if x != y {
        let pmf = dirac_pmf(x)
        discrete_pmf_mass_zero_outside[T](pmf, y)
        dirac_pmf_support(x)
        dirac_support_not_contains_of_not_eq(x, y)
        dirac_pmf(x).mass(y) = Real.0
    }
}

/// The expectation under a Dirac PMF is evaluation at its distinguished point.
theorem dirac_expectation[T](x: T, rv: T -> Real) {
    discrete_expectation(dirac_pmf(x), rv) = rv(x)
} by {
    let pmf = dirac_pmf(x)
    let f = mass_weighted_value(pmf.mass, rv)
    dirac_pmf_support(x)

    finite_set_sum_empty[T, Real](f)
    finite_set_sum(FiniteSet.empty[T], f) = Real.0

    finite_set_empty_contains_eq[T](x)
    finite_set_sum_insert[T, Real](FiniteSet.empty[T], x, f)
    finite_set_sum(fs_insert(FiniteSet.empty[T], x), f) = f(x) + finite_set_sum(FiniteSet.empty[T], f)

    dirac_pmf_mass_self(x)
    f(x) = Real.1 * rv(x)
    f(x) = rv(x)
    finite_set_sum(dirac_support(x), f) = rv(x)
}

/// The probability of an event under a Dirac PMF is the event indicator at the atom.
theorem dirac_event_prob_eq_indicator[T](x: T, event: FiniteSet[T]) {
    pmf_event_prob(dirac_pmf(x), event) = finite_event_indicator(event, x)
} by {
    discrete_expectation_indicator_eq_event_prob[T](dirac_pmf(x), event)
    dirac_expectation[T](x, finite_event_indicator(event))
}

/// The expectation of an event indicator under a Dirac PMF is the indicator at the atom.
theorem dirac_indicator_expectation[T](x: T, event: FiniteSet[T]) {
    discrete_expectation(dirac_pmf(x), finite_event_indicator(event)) = finite_event_indicator(event, x)
} by {
    discrete_expectation_indicator_eq_event_prob[T](dirac_pmf(x), event)
    dirac_event_prob_eq_indicator(x, event)
}
