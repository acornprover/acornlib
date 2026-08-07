from nat import Nat, from_nat
from list import List, map, sum, sum_map_of_pointwise
from real import Real, mul_one_over, gt_zero_imp_pos
from data.basic.set import list_set, list_set_contains_eq
from ordered_field import inverse_of_positive_is_positive
from finite_set import FiniteSet, fs_from_list, finite_set_subset_contains,
    finite_set_cardinality_has_exact_unique_list
from finite_set import finite_set_sum, finite_set_sum_eq_list_sum
from probability.discrete_pmf import DiscretePMF, is_pmf, pmf_event_prob,
    discrete_expectation, finite_event_indicator,
    finite_event_indicator_one_of_contains, finite_event_indicator_zero_of_not_contains,
    finite_event_indicator_nonneg, discrete_expectation_indicator_eq_event_prob,
    const_real_fn

/// The uniform mass attached to a finite support of cardinality `n`.
define finite_uniform_mass[T](support: FiniteSet[T], n: Nat, x: T) -> Real {
    finite_event_indicator(support, x) * (Real.1 / from_nat[Real](n))
}

/// The reciprocal of a positive real is positive.
lemma finite_uniform_real_one_div_pos(a: Real) {
    a > Real.0 implies Real.1 / a > Real.0
} by {
    if a > Real.0 {
        gt_zero_imp_pos(a)
        a.is_positive
        inverse_of_positive_is_positive(a)
        Real.0 < a.inverse
        a.inverse.is_positive
        Real.1.is_positive
        (Real.1 * a.inverse).is_positive
        Real.1 / a = Real.1 * a.inverse
        (Real.1 / a).is_positive
        Real.1 / a > Real.0
    }
}

/// On-support finite uniform mass is the reciprocal of the support cardinality.
theorem finite_uniform_mass_of_contains[T](support: FiniteSet[T], n: Nat, x: T) {
    support.contains(x) implies finite_uniform_mass(support, n, x) = Real.1 / from_nat[Real](n)
} by {
    if support.contains(x) {
        let c = Real.1 / from_nat[Real](n)
        finite_event_indicator_one_of_contains(support, x)
        finite_event_indicator(support, x) = Real.1
        finite_uniform_mass(support, n, x) = finite_event_indicator(support, x) * c
        finite_uniform_mass(support, n, x) = Real.1 * c
        Real.1 * c = c
        finite_uniform_mass(support, n, x) = c
        finite_uniform_mass(support, n, x) = Real.1 / from_nat[Real](n)
    }
    support.contains(x) implies finite_uniform_mass(support, n, x) = Real.1 / from_nat[Real](n)
}

/// Uniform mass is nonnegative everywhere when the real cardinality is positive.
theorem finite_uniform_mass_nonneg[T](support: FiniteSet[T], n: Nat, x: T) {
    from_nat[Real](n) > Real.0 implies Real.0 <= finite_uniform_mass(support, n, x)
} by {
    if from_nat[Real](n) > Real.0 {
        let c = Real.1 / from_nat[Real](n)
        finite_event_indicator_nonneg(support, x)
        Real.0 <= finite_event_indicator(support, x)
        finite_uniform_real_one_div_pos(from_nat[Real](n))
        c > Real.0
        Real.0 <= c
        if support.contains(x) {
            finite_event_indicator_one_of_contains(support, x)
            finite_event_indicator(support, x) = Real.1
            finite_uniform_mass(support, n, x) = finite_event_indicator(support, x) * c
            finite_uniform_mass(support, n, x) = Real.1 * c
            Real.1 * c = c
            finite_uniform_mass(support, n, x) = c
            Real.0 <= finite_uniform_mass(support, n, x)
        }
        if not support.contains(x) {
            finite_event_indicator_zero_of_not_contains(support, x)
            finite_event_indicator(support, x) = Real.0
            finite_uniform_mass(support, n, x) = finite_event_indicator(support, x) * c
            finite_uniform_mass(support, n, x) = Real.0 * c
            Real.0 * c = Real.0
            finite_uniform_mass(support, n, x) = Real.0
            Real.0 <= finite_uniform_mass(support, n, x)
        }
        support.contains(x) or not support.contains(x)
        Real.0 <= finite_uniform_mass(support, n, x)
    }
}

/// Uniform mass vanishes outside the finite support.
theorem finite_uniform_mass_zero_outside[T](support: FiniteSet[T], n: Nat, x: T) {
    not support.contains(x) implies finite_uniform_mass(support, n, x) = Real.0
} by {
    if not support.contains(x) {
        let c = Real.1 / from_nat[Real](n)
        finite_event_indicator_zero_of_not_contains(support, x)
        finite_event_indicator(support, x) = Real.0
        finite_uniform_mass(support, n, x) = finite_event_indicator(support, x) * c
        finite_uniform_mass(support, n, x) = Real.0 * c
        Real.0 * c = Real.0
        finite_uniform_mass(support, n, x) = Real.0
    }
}

/// Finite-set sums agree when the functions agree on the finite set.
lemma finite_uniform_set_sum_eq_of_contains_eq[T](s: FiniteSet[T], f: T -> Real, g: T -> Real) {
    (forall(x: T) { s.contains(x) implies f(x) = g(x) }) implies finite_set_sum(s, f) = finite_set_sum(s, g)
} by {
    if forall(x: T) { s.contains(x) implies f(x) = g(x) } {
        let items: List[T] satisfy {
            fs_from_list(items) = s and items.is_unique
        }
        forall(x: T) {
            if items.contains(x) {
                fs_from_list(items).underlying_set = list_set(items)
                list_set_contains_eq(items, x)
                list_set(items).contains(x) = items.contains(x)
                s.underlying_set.contains(x)
                s.contains(x)
                f(x) = g(x)
            }
        }
        sum_map_of_pointwise[T, Real](items, f, g)
        sum[Real](map(items, f)) = sum[Real](map(items, g))
        finite_set_sum_eq_list_sum[T, Real](items, f)
        finite_set_sum_eq_list_sum[T, Real](items, g)
        finite_set_sum(s, f) = sum[Real](map(items, f))
        finite_set_sum(s, g) = sum[Real](map(items, g))
        finite_set_sum(s, f) = finite_set_sum(s, g)
    }
}

/// The list sum of a constant real-valued map is length times that constant.
lemma list_sum_const_real_of_length[T](items: List[T], c: Real) {
    sum[Real](map(items, const_real_fn[T](c))) = from_nat[Real](items.length) * c
} by {
    define p(xs: List[T]) -> Bool {
        sum[Real](map(xs, const_real_fn[T](c))) = from_nat[Real](xs.length) * c
    }

    map[T, Real](List.nil[T], const_real_fn[T](c)) = List.nil[Real]
    sum[Real](List.nil[Real]) = Real.0
    from_nat[Real](List.nil[T].length) = from_nat[Real](Nat.0)
    from_nat[Real](Nat.0) = Real.0
    from_nat[Real](List.nil[T].length) * c = Real.0 * c
    Real.0 * c = Real.0
    p(List.nil[T])

    forall(head: T, tail: List[T]) {
        if p(tail) {
            map[T, Real](List.cons(head, tail), const_real_fn[T](c)) =
                List.cons(const_real_fn[T](c, head), map(tail, const_real_fn[T](c)))
            const_real_fn[T](c, head) = c
            map[T, Real](List.cons(head, tail), const_real_fn[T](c)) =
                List.cons(c, map(tail, const_real_fn[T](c)))
            sum[Real](List.cons(c, map(tail, const_real_fn[T](c)))) =
                c + sum[Real](map(tail, const_real_fn[T](c)))
            sum[Real](map(List.cons(head, tail), const_real_fn[T](c))) =
                c + sum[Real](map(tail, const_real_fn[T](c)))
            sum[Real](map(tail, const_real_fn[T](c))) = from_nat[Real](tail.length) * c
            sum[Real](map(List.cons(head, tail), const_real_fn[T](c))) =
                c + from_nat[Real](tail.length) * c

            List.cons(head, tail).length = tail.length.suc
            from_nat[Real](tail.length.suc) = from_nat[Real](tail.length) + Real.1
            from_nat[Real](List.cons(head, tail).length) = from_nat[Real](tail.length) + Real.1
            from_nat[Real](List.cons(head, tail).length) * c =
                (from_nat[Real](tail.length) + Real.1) * c
            (from_nat[Real](tail.length) + Real.1) * c =
                from_nat[Real](tail.length) * c + Real.1 * c
            Real.1 * c = c
            (from_nat[Real](tail.length) + Real.1) * c =
                from_nat[Real](tail.length) * c + c
            from_nat[Real](tail.length) * c + c = c + from_nat[Real](tail.length) * c
            from_nat[Real](List.cons(head, tail).length) * c =
                c + from_nat[Real](tail.length) * c
            sum[Real](map(List.cons(head, tail), const_real_fn[T](c))) =
                from_nat[Real](List.cons(head, tail).length) * c
            p(List.cons(head, tail))
        }
    }

    forall(head: T, tail: List[T]) {
        if p(tail) {
            p(List.cons(head, tail))
        }
        p(tail) implies p(List.cons(head, tail))
    }
    if exists(head: T, tail: List[T]) {
        p(tail) and not p(List.cons(head, tail))
    } {
        let (bad_head: T, bad_tail: List[T]) satisfy {
            p(bad_tail) and not p(List.cons(bad_head, bad_tail))
        }
        p(bad_tail)
        p(bad_tail) implies p(List.cons(bad_head, bad_tail))
        p(List.cons(bad_head, bad_tail))
        false
    }
    not exists(head: T, tail: List[T]) {
        p(tail) and not p(List.cons(head, tail))
    }
    List.induction(function(xs: List[T]) { p(xs) })
    function[T0](x0: List[T0] -> Bool, x1: List[T0]) {
        not x0(List.nil[T0]) or exists(k0: T0, k1: List[T0]) {
            x0(k1) and not x0(List.cons[T0](k0, k1))
        } or x0(x1)
    }[T](p, items)
    p(items)
}

/// The sum of a constant real over a finite set with cardinality `n`.
theorem finite_set_sum_const_real_of_cardinality_is[T](support: FiniteSet[T], c: Real, n: Nat) {
    support.cardinality_is(n) implies finite_set_sum(support, const_real_fn[T](c)) = from_nat[Real](n) * c
} by {
    if support.cardinality_is(n) {
        finite_set_cardinality_has_exact_unique_list(support, n)
        let items: List[T] satisfy {
            fs_from_list(items) = support and items.is_unique and items.length = n
        }
        finite_set_sum_eq_list_sum[T, Real](items, const_real_fn[T](c))
        finite_set_sum(fs_from_list(items), const_real_fn[T](c)) =
            sum[Real](map(items, const_real_fn[T](c)))
        list_sum_const_real_of_length(items, c)
        sum[Real](map(items, const_real_fn[T](c))) = from_nat[Real](items.length) * c
        items.length = n
        from_nat[Real](items.length) = from_nat[Real](n)
        finite_set_sum(support, const_real_fn[T](c)) = from_nat[Real](n) * c
    }
}

/// Summing the uniform mass over `support` gives one.
theorem finite_uniform_mass_total[T](support: FiniteSet[T], n: Nat) {
    support.cardinality_is(n) and from_nat[Real](n) > Real.0 implies
        finite_set_sum(support, finite_uniform_mass(support, n)) = Real.1
} by {
    if support.cardinality_is(n) and from_nat[Real](n) > Real.0 {
        let c = Real.1 / from_nat[Real](n)
        forall(x: T) {
            if support.contains(x) {
                finite_uniform_mass_of_contains(support, n, x)
                finite_uniform_mass(support, n, x) = c
                const_real_fn[T](c, x) = c
                finite_uniform_mass(support, n, x) = const_real_fn[T](c, x)
            }
        }
        finite_uniform_set_sum_eq_of_contains_eq(support, finite_uniform_mass(support, n), const_real_fn[T](c))
        finite_set_sum(support, finite_uniform_mass(support, n)) = finite_set_sum(support, const_real_fn[T](c))
        finite_set_sum_const_real_of_cardinality_is(support, c, n)
        finite_set_sum(support, const_real_fn[T](c)) = from_nat[Real](n) * c
        c = Real.1 / from_nat[Real](n)
        from_nat[Real](n) != Real.0
        mul_one_over(from_nat[Real](n), from_nat[Real](n))
        from_nat[Real](n) * (Real.1 / from_nat[Real](n)) = from_nat[Real](n) / from_nat[Real](n)
        from_nat[Real](n) / from_nat[Real](n) = Real.1
        from_nat[Real](n) * c = Real.1
        finite_set_sum(support, finite_uniform_mass(support, n)) = Real.1
    }
}

/// Uniform mass data is a probability mass function.
theorem finite_uniform_is_pmf[T](support: FiniteSet[T], n: Nat) {
    support.cardinality_is(n) and from_nat[Real](n) > Real.0 implies is_pmf(support, finite_uniform_mass(support, n))
} by {
    if support.cardinality_is(n) and from_nat[Real](n) > Real.0 {
        let cond_nonneg: Bool = forall(x: T) { Real.0 <= finite_uniform_mass(support, n, x) }
        let cond_zero: Bool = forall(x: T) {
            not support.contains(x) implies finite_uniform_mass(support, n, x) = Real.0
        }
        let cond_sum: Bool = finite_set_sum(support, finite_uniform_mass(support, n)) = Real.1

        forall(x: T) {
            finite_uniform_mass_nonneg(support, n, x)
            Real.0 <= finite_uniform_mass(support, n, x)
        }
        cond_nonneg

        forall(x: T) {
            finite_uniform_mass_zero_outside(support, n, x)
        }
        cond_zero

        finite_uniform_mass_total(support, n)
        cond_sum

        cond_nonneg and cond_zero
        cond_nonneg and cond_zero and cond_sum
        is_pmf(support, finite_uniform_mass(support, n)) = (cond_nonneg and cond_zero and cond_sum)
        is_pmf(support, finite_uniform_mass(support, n))
    }
}

/// A bundled uniform PMF exists for every finite support with positive real cardinality.
theorem finite_uniform_pmf_exists[T](support: FiniteSet[T], n: Nat) {
    support.cardinality_is(n) and from_nat[Real](n) > Real.0 implies exists(pmf: DiscretePMF[T]) {
        DiscretePMF[T].new(support, finite_uniform_mass(support, n)) = Option.some(pmf)
    }
} by {
    if support.cardinality_is(n) and from_nat[Real](n) > Real.0 {
        finite_uniform_is_pmf(support, n)
        is_pmf(support, finite_uniform_mass(support, n))
        exists(pmf: DiscretePMF[T]) {
            DiscretePMF[T].new(support, finite_uniform_mass(support, n)) = Option.some(pmf)
        }
    }
}

/// Event probability is the cardinality ratio when event masses agree with finite uniform data.
theorem finite_uniform_event_prob_eq_cardinality_ratio[T](
    support: FiniteSet[T],
    event: FiniteSet[T],
    n: Nat,
    k: Nat,
    pmf: DiscretePMF[T]
) {
    support.cardinality_is(n) and event.cardinality_is(k) and event.subset_eq(support) and
        from_nat[Real](n) > Real.0 and
        (forall(x: T) { event.contains(x) implies pmf.mass(x) = finite_uniform_mass(support, n, x) })
        implies pmf_event_prob(pmf, event) = from_nat[Real](k) / from_nat[Real](n)
} by {
    if support.cardinality_is(n) and event.cardinality_is(k) and event.subset_eq(support) and
        from_nat[Real](n) > Real.0 and
        (forall(x: T) { event.contains(x) implies pmf.mass(x) = finite_uniform_mass(support, n, x) }) {
        let c = Real.1 / from_nat[Real](n)
        forall(x: T) {
            if event.contains(x) {
                event.subset_eq(support) and event.contains(x)
                finite_set_subset_contains(event, support, x)
                support.contains(x)
                finite_uniform_mass_of_contains(support, n, x)
                finite_uniform_mass(support, n, x) = c
                pmf.mass(x) = finite_uniform_mass(support, n, x)
                pmf.mass(x) = c
                const_real_fn[T](c, x) = c
                pmf.mass(x) = const_real_fn[T](c, x)
            }
        }
        pmf_event_prob(pmf, event) = finite_set_sum(event, pmf.mass)
        finite_uniform_set_sum_eq_of_contains_eq(event, pmf.mass, const_real_fn[T](c))
        finite_set_sum(event, pmf.mass) = finite_set_sum(event, const_real_fn[T](c))
        finite_set_sum_const_real_of_cardinality_is(event, c, k)
        finite_set_sum(event, const_real_fn[T](c)) = from_nat[Real](k) * c
        c = Real.1 / from_nat[Real](n)
        mul_one_over(from_nat[Real](k), from_nat[Real](n))
        from_nat[Real](k) * (Real.1 / from_nat[Real](n)) = from_nat[Real](k) / from_nat[Real](n)
        from_nat[Real](k) * c = from_nat[Real](k) / from_nat[Real](n)
        pmf_event_prob(pmf, event) = from_nat[Real](k) / from_nat[Real](n)
    }
}

/// Indicator expectation is the cardinality ratio when event masses agree with finite uniform data.
theorem finite_uniform_indicator_expectation_eq_cardinality_ratio[T](
    support: FiniteSet[T],
    event: FiniteSet[T],
    n: Nat,
    k: Nat,
    pmf: DiscretePMF[T]
) {
    support.cardinality_is(n) and event.cardinality_is(k) and event.subset_eq(support) and
        from_nat[Real](n) > Real.0 and
        (forall(x: T) { event.contains(x) implies pmf.mass(x) = finite_uniform_mass(support, n, x) })
        implies discrete_expectation(pmf, finite_event_indicator(event)) =
            from_nat[Real](k) / from_nat[Real](n)
} by {
    if support.cardinality_is(n) and event.cardinality_is(k) and event.subset_eq(support) and
        from_nat[Real](n) > Real.0 and
        (forall(x: T) { event.contains(x) implies pmf.mass(x) = finite_uniform_mass(support, n, x) }) {
        discrete_expectation_indicator_eq_event_prob(pmf, event)
        discrete_expectation(pmf, finite_event_indicator(event)) = pmf_event_prob(pmf, event)
        finite_uniform_event_prob_eq_cardinality_ratio(support, event, n, k, pmf)
        pmf_event_prob(pmf, event) = from_nat[Real](k) / from_nat[Real](n)
        discrete_expectation(pmf, finite_event_indicator(event)) =
            from_nat[Real](k) / from_nat[Real](n)
    }
}
