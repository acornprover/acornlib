from nat import Nat, from_nat, from_nat_one, pow_zero, pow_one, one_pow, alt_induction,
    lt_or_lte, lt_suc, lt_and_lte
from real import Real, mul_nonneg
from order import lt_imp_lte
from ordered_field import zero_is_smaller_than_one
from finite_set import FiniteSet, finite_set_sum
from probability.bernoulli_pmf import one_sub_nonneg
from probability.discrete_pmf import DiscretePMF, is_pmf, mass_weighted_value,
    square_fn, square_dev, discrete_expectation, discrete_variance
from combinatorics import binom, choose_zero, choose_n, choose_one, choose_out_of_bounds
from data.nat.nat_range_set import range_set, range_set_contains_eq
from data.nat.nat_range_sum import range_sum, range_sum_zero, range_sum_suc
from data.nat.nat_range_sum_bridge import range_sum_eq_partial, range_sum_eq_finite_set_sum
from comm_ring import binomial, binomial_term
from list import partial, partial_pointwise_eq
from data.basic.functions import function_extensionality
from algebra.add_ordered_group import add_le_add
from algebra.ring.ring import mul_neg_neg

numerals Nat

/// The probability mass of the Binomial(n, p) distribution at `k`:
/// C(n,k) p^k (1-p)^(n-k) for k = 0, ..., n.
define binomial_mass(n: Nat, p: Real, k: Nat) -> Real {
    from_nat[Real](n.binom(k)) * p.pow(k) * (Real.1 - p).pow(n - k)
}

/// The finite support `{0, ..., n}` of the Binomial(n, p) distribution.
define binomial_support(n: Nat) -> FiniteSet[Nat] {
    range_set(n.suc)
}

/// The identity random variable `X(k) = k` on the naturals, as a real value.
define nat_to_real(k: Nat) -> Real {
    from_nat[Real](k)
}

/// The real value of any natural number is nonnegative.
theorem from_nat_nonneg_real(n: Nat) {
    Real.0 <= from_nat[Real](n)
} by {
    define p(x: Nat) -> Bool {
        Real.0 <= from_nat[Real](x)
    }
    from_nat[Real](Nat.0) = Real.0
    p(Nat.0)
    forall(x: Nat) {
        if p(x) {
            from_nat[Real](x.suc) = from_nat[Real](x) + Real.1
            zero_is_smaller_than_one[Real]
            lt_imp_lte[Real](Real.0, Real.1)
            Real.0 <= Real.1
            add_le_add[Real](Real.0, from_nat[Real](x), Real.0, Real.1)
            Real.0 + Real.0 <= from_nat[Real](x) + Real.1
            Real.0 + Real.0 = Real.0
            Real.0 <= from_nat[Real](x) + Real.1
            Real.0 <= from_nat[Real](x.suc)
            p(x.suc)
        }
    }
    p(Nat.0) and forall(x: Nat) { p(x) implies p(x.suc) }
    alt_induction(p)
    forall(x: Nat) { p(x) }
    p(n)
}

/// `a >= 0` is the same as `0 <= a`.
theorem gte_zero_unfold(a: Real) {
    a >= Real.0 = (Real.0 <= a)
} by {
    a >= Real.0 = (Real.0 <= a)
}

/// Every power of a nonnegative real is nonnegative.
theorem real_pow_nonneg_of_nonneg(r: Real, k: Nat) {
    Real.0 <= r implies Real.0 <= r.pow(k)
} by {
    if Real.0 <= r {
        define p(x: Nat) -> Bool {
            Real.0 <= r.pow(x)
        }
        pow_zero[Real](r)
        r.pow(0) = Real.1
        zero_is_smaller_than_one[Real]
        lt_imp_lte[Real](Real.0, Real.1)
        Real.0 <= Real.1
        p(Nat.0)
        forall(x: Nat) {
            if p(x) {
                r.pow(x.suc) = r * r.pow(x)
                gte_zero_unfold(r)
                r >= Real.0
                gte_zero_unfold(r.pow(x))
                r.pow(x) >= Real.0
                mul_nonneg(r, r.pow(x))
                r * r.pow(x) >= Real.0
                gte_zero_unfold(r * r.pow(x))
                Real.0 <= r * r.pow(x)
                Real.0 <= r.pow(x.suc)
                p(x.suc)
            }
        }
        p(Nat.0) and forall(x: Nat) { p(x) implies p(x.suc) }
        alt_induction(p)
        forall(x: Nat) { p(x) }
        p(k)
        p(k) = (Real.0 <= r.pow(k))
        Real.0 <= r.pow(k)
    }
}

/// The real value of the natural number two is `1 + 1`.
theorem from_nat_two_real {
    from_nat[Real](Nat.2) = Real.1 + Real.1
} by {
    from_nat[Real](Nat.1.suc) = from_nat[Real](Nat.1) + Real.1
    from_nat[Real](Nat.1) = from_nat[Real](Nat.0) + Real.1
    from_nat[Real](Nat.0) = Real.0
    from_nat[Real](Nat.1) = Real.0 + Real.1
    Real.0 + Real.1 = Real.1
    from_nat[Real](Nat.1) = Real.1
    from_nat[Real](Nat.2) = Real.1 + Real.1
}

/// `p + (1 - p) = 1` for every real `p`.
theorem real_p_add_one_sub(p: Real) {
    p + (Real.1 - p) = Real.1
} by {
    p + (Real.1 - p) = p + (Real.1 + -p)
    p + (Real.1 + -p) = (p + Real.1) + -p
    (p + Real.1) + -p = (Real.1 + p) + -p
    (Real.1 + p) + -p = Real.1 + (p + -p)
    Real.1 + (p + -p) = Real.1 + Real.0
    Real.1 + Real.0 = Real.1
}

/// The binomial mass is exactly the k-th term of the real binomial expansion
/// of (p + (1-p))^n.
theorem binomial_mass_eq_binomial_term(n: Nat, p: Real, k: Nat) {
    binomial_mass(n, p, k) = binomial_term[Real](p, Real.1 - p, n, k)
} by {
    binomial_mass(n, p, k) = from_nat[Real](n.binom(k)) * p.pow(k) * (Real.1 - p).pow(n - k)
    binomial_term[Real](p, Real.1 - p, n, k) =
        from_nat[Real](n.binom(k)) * p.pow(k) * (Real.1 - p).pow(n - k)
}

/// The binomial mass function is the real binomial term function for (p + (1-p))^n.
theorem binomial_mass_fn_eq_binomial_term(n: Nat, p: Real) {
    binomial_mass(n, p) = binomial_term[Real](p, Real.1 - p, n)
} by {
    forall(k: Nat) {
        binomial_mass_eq_binomial_term(n, p, k)
        binomial_mass(n, p, k) = binomial_term[Real](p, Real.1 - p, n, k)
    }
    function_extensionality[Nat, Real](binomial_mass(n, p), binomial_term[Real](p, Real.1 - p, n))
}

/// The binomial probabilities sum to one:
/// Σ_{k=0}^{n} C(n,k) p^k (1-p)^(n-k) = 1, by the binomial theorem
/// applied to (p + (1-p))^n = 1^n.
theorem binomial_mass_total(n: Nat, p: Real) {
    finite_set_sum(binomial_support(n), binomial_mass(n, p)) = Real.1
} by {
    range_sum_eq_finite_set_sum(binomial_mass(n, p), n.suc)
    finite_set_sum(binomial_support(n), binomial_mass(n, p)) =
        range_sum(binomial_mass(n, p), n.suc)
    range_sum_eq_partial(binomial_mass(n, p), n.suc)
    range_sum(binomial_mass(n, p), n.suc) = partial(binomial_mass(n, p), n.suc)
    finite_set_sum(binomial_support(n), binomial_mass(n, p)) =
        partial(binomial_mass(n, p), n.suc)

    forall(k: Nat) {
        binomial_mass_eq_binomial_term(n, p, k)
        binomial_mass(n, p, k) = binomial_term[Real](p, Real.1 - p, n, k)
    }
    forall(k: Nat) {
        if k < n.suc {
            binomial_mass_eq_binomial_term(n, p, k)
            binomial_mass(n, p, k) = binomial_term[Real](p, Real.1 - p, n, k)
        }
    }
    partial_pointwise_eq[Real](binomial_mass(n, p), binomial_term[Real](p, Real.1 - p, n), n.suc)
    partial(binomial_mass(n, p), n.suc) =
        partial(binomial_term[Real](p, Real.1 - p, n), n.suc)

    binomial[Real](p, Real.1 - p, n)
    (p + (Real.1 - p)).pow(n) = partial(binomial_term[Real](p, Real.1 - p, n), n.suc)
    partial(binomial_mass(n, p), n.suc) = (p + (Real.1 - p)).pow(n)
    real_p_add_one_sub(p)
    p + (Real.1 - p) = Real.1
    (p + (Real.1 - p)).pow(n) = Real.1.pow(n)
    one_pow[Real](n)
    Real.1.pow(n) = Real.1
    finite_set_sum(binomial_support(n), binomial_mass(n, p)) = Real.1
}

/// The probability of no successes: P(X = 0) = (1-p)^n.
theorem binomial_mass_zero(n: Nat, p: Real) {
    binomial_mass(n, p, Nat.0) = (Real.1 - p).pow(n)
} by {
    binomial_mass(n, p, Nat.0) = from_nat[Real](n.binom(0)) * p.pow(0) * (Real.1 - p).pow(n - 0)
    choose_zero(n)
    n.binom(0) = Nat.1
    from_nat[Real](n.binom(0)) = from_nat[Real](Nat.1)
    from_nat_one[Real]
    from_nat[Real](Nat.1) = Real.1
    from_nat[Real](n.binom(0)) = Real.1
    pow_zero[Real](p)
    p.pow(0) = Real.1
    n - 0 = n
    (Real.1 - p).pow(n - 0) = (Real.1 - p).pow(n)
    Real.1 * Real.1 * (Real.1 - p).pow(n) = (Real.1 - p).pow(n)
    binomial_mass(n, p, Nat.0) = (Real.1 - p).pow(n)
}

/// The probability of n successes: P(X = n) = p^n.
theorem binomial_mass_n(n: Nat, p: Real) {
    binomial_mass(n, p, n) = p.pow(n)
} by {
    binomial_mass(n, p, n) = from_nat[Real](n.binom(n)) * p.pow(n) * (Real.1 - p).pow(n - n)
    choose_n(n)
    n.binom(n) = Nat.1
    from_nat[Real](n.binom(n)) = from_nat[Real](Nat.1)
    from_nat_one[Real]
    from_nat[Real](Nat.1) = Real.1
    from_nat[Real](n.binom(n)) = Real.1
    n - n = Nat.0
    pow_zero[Real](Real.1 - p)
    (Real.1 - p).pow(0) = Real.1
    (Real.1 - p).pow(n - n) = Real.1
    Real.1 * p.pow(n) * Real.1 = p.pow(n)
    binomial_mass(n, p, n) = p.pow(n)
}

/// The binomial mass function is nonnegative when `0 <= p <= 1`.
theorem binomial_mass_nonneg(n: Nat, p: Real, k: Nat) {
    Real.0 <= p and p <= Real.1 implies Real.0 <= binomial_mass(n, p, k)
} by {
    if Real.0 <= p and p <= Real.1 {
        from_nat_nonneg_real(n.binom(k))
        Real.0 <= from_nat[Real](n.binom(k))
        real_pow_nonneg_of_nonneg(p, k)
        Real.0 <= p.pow(k)
        one_sub_nonneg(p)
        Real.0 <= Real.1 - p
        real_pow_nonneg_of_nonneg(Real.1 - p, n - k)
        Real.0 <= (Real.1 - p).pow(n - k)
        mul_nonneg(from_nat[Real](n.binom(k)), p.pow(k))
        gte_zero_unfold(from_nat[Real](n.binom(k)))
        from_nat[Real](n.binom(k)) >= Real.0
        gte_zero_unfold(p.pow(k))
        p.pow(k) >= Real.0
        from_nat[Real](n.binom(k)) * p.pow(k) >= Real.0
        gte_zero_unfold(from_nat[Real](n.binom(k)) * p.pow(k))
        Real.0 <= from_nat[Real](n.binom(k)) * p.pow(k)
        mul_nonneg(from_nat[Real](n.binom(k)) * p.pow(k), (Real.1 - p).pow(n - k))
        gte_zero_unfold((Real.1 - p).pow(n - k))
        (Real.1 - p).pow(n - k) >= Real.0
        from_nat[Real](n.binom(k)) * p.pow(k) * (Real.1 - p).pow(n - k) >= Real.0
        gte_zero_unfold(from_nat[Real](n.binom(k)) * p.pow(k) * (Real.1 - p).pow(n - k))
        Real.0 <= from_nat[Real](n.binom(k)) * p.pow(k) * (Real.1 - p).pow(n - k)
        Real.0 <= binomial_mass(n, p, k)
    }
}

/// The binomial mass vanishes outside the support `{0, ..., n}`.
theorem binomial_mass_zero_outside(n: Nat, p: Real, k: Nat) {
    not binomial_support(n).contains(k) implies binomial_mass(n, p, k) = Real.0
} by {
    if not binomial_support(n).contains(k) {
        binomial_support(n) = range_set(n.suc)
        binomial_support(n).contains(k) = range_set(n.suc).contains(k)
        range_set_contains_eq(n.suc, k)
        range_set(n.suc).contains(k) = (k < n.suc)
        binomial_support(n).contains(k) = (k < n.suc)
        not (k < n.suc)
        lt_or_lte(k, n.suc)
        k < n.suc or n.suc <= k
        n.suc <= k
        lt_suc(n)
        n < n.suc
        lt_and_lte(n, n.suc, k)
        n < k
        choose_out_of_bounds(n, k)
        n.binom(k) = Nat.0
        from_nat[Real](n.binom(k)) = from_nat[Real](Nat.0)
        from_nat[Real](Nat.0) = Real.0
        from_nat[Real](n.binom(k)) = Real.0
        Real.0 * p.pow(k) * (Real.1 - p).pow(n - k) = Real.0
        binomial_mass(n, p, k) = Real.0
    }
}

/// For `0 <= p <= 1`, the binomial mass data is a valid probability mass function.
theorem binomial_is_pmf(n: Nat, p: Real) {
    Real.0 <= p and p <= Real.1 implies is_pmf(binomial_support(n), binomial_mass(n, p))
} by {
    if Real.0 <= p and p <= Real.1 {
        let cond_nonneg: Bool = forall(x: Nat) { Real.0 <= binomial_mass(n, p, x) }
        let cond_zero: Bool = forall(x: Nat) {
            not binomial_support(n).contains(x) implies binomial_mass(n, p, x) = Real.0
        }
        let cond_sum: Bool = finite_set_sum(binomial_support(n), binomial_mass(n, p)) = Real.1

        forall(x: Nat) {
            binomial_mass_nonneg(n, p, x)
            Real.0 <= binomial_mass(n, p, x)
        }
        cond_nonneg

        forall(x: Nat) {
            binomial_mass_zero_outside(n, p, x)
        }
        cond_zero

        binomial_mass_total(n, p)
        cond_sum

        cond_nonneg and cond_zero
        cond_nonneg and cond_zero and cond_sum
        is_pmf(binomial_support(n), binomial_mass(n, p)) = (cond_nonneg and cond_zero and cond_sum)
        is_pmf(binomial_support(n), binomial_mass(n, p))
    }
}

/// A bundled Binomial PMF exists for every parameter in `[0, 1]`.
theorem binomial_pmf_exists(n: Nat, p: Real) {
    Real.0 <= p and p <= Real.1 implies exists(pmf: DiscretePMF[Nat]) {
        DiscretePMF[Nat].new(binomial_support(n), binomial_mass(n, p)) = Option.some(pmf)
    }
} by {
    if Real.0 <= p and p <= Real.1 {
        binomial_is_pmf(n, p)
        is_pmf(binomial_support(n), binomial_mass(n, p))
        exists(pmf: DiscretePMF[Nat]) {
            DiscretePMF[Nat].new(binomial_support(n), binomial_mass(n, p)) = Option.some(pmf)
        }
    }
}

/// The Binomial(1, p) mass at k = 0 is `1 - p`.
theorem binomial_mass_one_zero(p: Real) {
    binomial_mass(Nat.1, p, Nat.0) = Real.1 - p
} by {
    binomial_mass(Nat.1, p, Nat.0) =
        from_nat[Real](Nat.1.binom(0)) * p.pow(0) * (Real.1 - p).pow(Nat.1 - 0)
    choose_zero(Nat.1)
    Nat.1.binom(0) = Nat.1
    from_nat[Real](Nat.1.binom(0)) = from_nat[Real](Nat.1)
    from_nat_one[Real]
    from_nat[Real](Nat.1) = Real.1
    from_nat[Real](Nat.1.binom(0)) = Real.1
    pow_zero[Real](p)
    p.pow(0) = Real.1
    Nat.1 - 0 = Nat.1
    (Real.1 - p).pow(Nat.1 - 0) = (Real.1 - p).pow(Nat.1)
    pow_one[Real](Real.1 - p)
    (Real.1 - p).pow(Nat.1) = Real.1 - p
    Real.1 * Real.1 * (Real.1 - p) = Real.1 - p
    binomial_mass(Nat.1, p, Nat.0) = Real.1 - p
}

/// The Binomial(1, p) mass at k = 1 is `p`.
theorem binomial_mass_one_one(p: Real) {
    binomial_mass(Nat.1, p, Nat.1) = p
} by {
    binomial_mass(Nat.1, p, Nat.1) =
        from_nat[Real](Nat.1.binom(1)) * p.pow(1) * (Real.1 - p).pow(Nat.1 - 1)
    choose_n(Nat.1)
    Nat.1.binom(1) = Nat.1
    from_nat[Real](Nat.1.binom(1)) = from_nat[Real](Nat.1)
    from_nat_one[Real]
    from_nat[Real](Nat.1) = Real.1
    from_nat[Real](Nat.1.binom(1)) = Real.1
    pow_one[Real](p)
    p.pow(1) = p
    Nat.1 - 1 = Nat.0
    pow_zero[Real](Real.1 - p)
    (Real.1 - p).pow(0) = Real.1
    (Real.1 - p).pow(Nat.1 - 1) = Real.1
    Real.1 * p * Real.1 = p
    binomial_mass(Nat.1, p, Nat.1) = p
}

/// The Binomial(2, p) mass at k = 1 is `2 p (1 - p)`.
theorem binomial_mass_two_one(p: Real) {
    binomial_mass(Nat.2, p, Nat.1) = (Real.1 + Real.1) * p * (Real.1 - p)
} by {
    binomial_mass(Nat.2, p, Nat.1) =
        from_nat[Real](Nat.2.binom(1)) * p.pow(1) * (Real.1 - p).pow(Nat.2 - 1)
    choose_one(Nat.2)
    Nat.2.binom(1) = Nat.2
    from_nat[Real](Nat.2.binom(1)) = from_nat[Real](Nat.2)
    from_nat_two_real
    from_nat[Real](Nat.2) = Real.1 + Real.1
    from_nat[Real](Nat.2.binom(1)) = Real.1 + Real.1
    pow_one[Real](p)
    p.pow(1) = p
    Nat.2 - 1 = Nat.1
    (Real.1 - p).pow(Nat.2 - 1) = (Real.1 - p).pow(Nat.1)
    pow_one[Real](Real.1 - p)
    (Real.1 - p).pow(Nat.1) = Real.1 - p
    binomial_mass(Nat.2, p, Nat.1) = (Real.1 + Real.1) * p * (Real.1 - p)
}

/// The Binomial(2, p) mass at k = 2 is `p^2`.
theorem binomial_mass_two_two(p: Real) {
    binomial_mass(Nat.2, p, Nat.2) = p * p
} by {
    binomial_mass(Nat.2, p, Nat.2) =
        from_nat[Real](Nat.2.binom(2)) * p.pow(2) * (Real.1 - p).pow(Nat.2 - 2)
    choose_n(Nat.2)
    Nat.2.binom(2) = Nat.1
    from_nat[Real](Nat.2.binom(2)) = from_nat[Real](Nat.1)
    from_nat_one[Real]
    from_nat[Real](Nat.1) = Real.1
    from_nat[Real](Nat.2.binom(2)) = Real.1
    p.pow(Nat.2) = p * p
    Nat.2 - 2 = Nat.0
    pow_zero[Real](Real.1 - p)
    (Real.1 - p).pow(0) = Real.1
    (Real.1 - p).pow(Nat.2 - 2) = Real.1
    Real.1 * (p * p) * Real.1 = p * p
    binomial_mass(Nat.2, p, Nat.2) = p * p
}

/// The expectation of the Binomial(1, p) count is `p`:
/// Σ_{k=0}^{1} k·C(1,k) p^k (1-p)^(1-k) = p.
theorem binomial_expectation_one(p: Real) {
    finite_set_sum(binomial_support(Nat.1),
        mass_weighted_value(binomial_mass(Nat.1, p), nat_to_real)) = p
} by {
    let g = mass_weighted_value(binomial_mass(Nat.1, p), nat_to_real)
    range_sum_eq_finite_set_sum(g, Nat.2)
    finite_set_sum(binomial_support(Nat.1), g) = range_sum(g, Nat.2)
    range_sum_suc(g, Nat.1)
    range_sum(g, Nat.2) = range_sum(g, Nat.1) + g(Nat.1)
    range_sum_suc(g, Nat.0)
    range_sum(g, Nat.1) = range_sum(g, Nat.0) + g(Nat.0)
    range_sum_zero(g)
    range_sum(g, Nat.0) = Real.0
    range_sum(g, Nat.2) = Real.0 + g(Nat.0) + g(Nat.1)
    g(Nat.0) = binomial_mass(Nat.1, p, Nat.0) * nat_to_real(Nat.0)
    nat_to_real(Nat.0) = from_nat[Real](Nat.0)
    from_nat[Real](Nat.0) = Real.0
    g(Nat.0) = binomial_mass(Nat.1, p, Nat.0) * Real.0
    binomial_mass(Nat.1, p, Nat.0) * Real.0 = Real.0
    g(Nat.0) = Real.0
    g(Nat.1) = binomial_mass(Nat.1, p, Nat.1) * nat_to_real(Nat.1)
    nat_to_real(Nat.1) = from_nat[Real](Nat.1)
    from_nat[Real](Nat.1) = Real.1
    g(Nat.1) = binomial_mass(Nat.1, p, Nat.1) * Real.1
    binomial_mass_one_one(p)
    binomial_mass(Nat.1, p, Nat.1) = p
    g(Nat.1) = p * Real.1
    p * Real.1 = p
    g(Nat.1) = p
    Real.0 + Real.0 + p = p
    finite_set_sum(binomial_support(Nat.1), g) = p
}

/// The second moment of the Binomial(1, p) count is `p`:
/// Σ_{k=0}^{1} k^2·C(1,k) p^k (1-p)^(1-k) = p.
theorem binomial_second_moment_one(p: Real) {
    finite_set_sum(binomial_support(Nat.1),
        mass_weighted_value(binomial_mass(Nat.1, p), square_fn(nat_to_real))) = p
} by {
    let h = mass_weighted_value(binomial_mass(Nat.1, p), square_fn(nat_to_real))
    range_sum_eq_finite_set_sum(h, Nat.2)
    finite_set_sum(binomial_support(Nat.1), h) = range_sum(h, Nat.2)
    range_sum_suc(h, Nat.1)
    range_sum(h, Nat.2) = range_sum(h, Nat.1) + h(Nat.1)
    range_sum_suc(h, Nat.0)
    range_sum(h, Nat.1) = range_sum(h, Nat.0) + h(Nat.0)
    range_sum_zero(h)
    range_sum(h, Nat.0) = Real.0
    range_sum(h, Nat.2) = Real.0 + h(Nat.0) + h(Nat.1)
    h(Nat.0) = binomial_mass(Nat.1, p, Nat.0) * square_fn(nat_to_real, Nat.0)
    square_fn(nat_to_real, Nat.0) = nat_to_real(Nat.0) * nat_to_real(Nat.0)
    nat_to_real(Nat.0) = from_nat[Real](Nat.0)
    from_nat[Real](Nat.0) = Real.0
    square_fn(nat_to_real, Nat.0) = Real.0 * Real.0
    Real.0 * Real.0 = Real.0
    square_fn(nat_to_real, Nat.0) = Real.0
    h(Nat.0) = binomial_mass(Nat.1, p, Nat.0) * Real.0
    binomial_mass(Nat.1, p, Nat.0) * Real.0 = Real.0
    h(Nat.0) = Real.0
    h(Nat.1) = binomial_mass(Nat.1, p, Nat.1) * square_fn(nat_to_real, Nat.1)
    square_fn(nat_to_real, Nat.1) = nat_to_real(Nat.1) * nat_to_real(Nat.1)
    nat_to_real(Nat.1) = from_nat[Real](Nat.1)
    from_nat[Real](Nat.1) = Real.1
    square_fn(nat_to_real, Nat.1) = Real.1 * Real.1
    Real.1 * Real.1 = Real.1
    square_fn(nat_to_real, Nat.1) = Real.1
    binomial_mass_one_one(p)
    binomial_mass(Nat.1, p, Nat.1) = p
    h(Nat.1) = p * Real.1
    p * Real.1 = p
    h(Nat.1) = p
    Real.0 + Real.0 + p = p
    finite_set_sum(binomial_support(Nat.1), h) = p
}

/// The variance of the Binomial(1, p) count is `p (1 - p)`:
/// Σ_{k=0}^{1} (k - p)^2·C(1,k) p^k (1-p)^(1-k) = p (1 - p).
theorem binomial_variance_one(p: Real) {
    finite_set_sum(binomial_support(Nat.1),
        mass_weighted_value(binomial_mass(Nat.1, p), square_dev(nat_to_real, p)))
        = p * (Real.1 - p)
} by {
    let v = mass_weighted_value(binomial_mass(Nat.1, p), square_dev(nat_to_real, p))
    range_sum_eq_finite_set_sum(v, Nat.2)
    finite_set_sum(binomial_support(Nat.1), v) = range_sum(v, Nat.2)
    range_sum_suc(v, Nat.1)
    range_sum(v, Nat.2) = range_sum(v, Nat.1) + v(Nat.1)
    range_sum_suc(v, Nat.0)
    range_sum(v, Nat.1) = range_sum(v, Nat.0) + v(Nat.0)
    range_sum_zero(v)
    range_sum(v, Nat.0) = Real.0
    range_sum(v, Nat.2) = Real.0 + v(Nat.0) + v(Nat.1)
    v(Nat.0) = binomial_mass(Nat.1, p, Nat.0) * square_dev(nat_to_real, p, Nat.0)
    square_dev(nat_to_real, p, Nat.0) = (nat_to_real(Nat.0) - p) * (nat_to_real(Nat.0) - p)
    nat_to_real(Nat.0) = from_nat[Real](Nat.0)
    from_nat[Real](Nat.0) = Real.0
    nat_to_real(Nat.0) - p = Real.0 - p
    (nat_to_real(Nat.0) - p) * (nat_to_real(Nat.0) - p) = (Real.0 - p) * (Real.0 - p)
    Real.0 - p = -p
    (Real.0 - p) * (Real.0 - p) = (-p) * (-p)
    mul_neg_neg[Real](p, p)
    (-p) * (-p) = p * p
    (nat_to_real(Nat.0) - p) * (nat_to_real(Nat.0) - p) = p * p
    square_dev(nat_to_real, p, Nat.0) = p * p
    binomial_mass_one_zero(p)
    binomial_mass(Nat.1, p, Nat.0) = Real.1 - p
    v(Nat.0) = (Real.1 - p) * (p * p)
    v(Nat.1) = binomial_mass(Nat.1, p, Nat.1) * square_dev(nat_to_real, p, Nat.1)
    square_dev(nat_to_real, p, Nat.1) = (nat_to_real(Nat.1) - p) * (nat_to_real(Nat.1) - p)
    nat_to_real(Nat.1) = from_nat[Real](Nat.1)
    from_nat[Real](Nat.1) = Real.1
    square_dev(nat_to_real, p, Nat.1) = (Real.1 - p) * (Real.1 - p)
    binomial_mass_one_one(p)
    binomial_mass(Nat.1, p, Nat.1) = p
    v(Nat.1) = p * ((Real.1 - p) * (Real.1 - p))
    (Real.1 - p) * (p * p) = (p * (Real.1 - p)) * p
    p * ((Real.1 - p) * (Real.1 - p)) = (p * (Real.1 - p)) * (Real.1 - p)
    (p * (Real.1 - p)) * p + (p * (Real.1 - p)) * (Real.1 - p) = (p * (Real.1 - p)) * (p + (Real.1 - p))
    real_p_add_one_sub(p)
    p + (Real.1 - p) = Real.1
    (p * (Real.1 - p)) * (p + (Real.1 - p)) = (p * (Real.1 - p)) * Real.1
    (p * (Real.1 - p)) * Real.1 = p * (Real.1 - p)
    (Real.1 - p) * (p * p) + p * ((Real.1 - p) * (Real.1 - p)) = p * (Real.1 - p)
    Real.0 + v(Nat.0) + v(Nat.1) = p * (Real.1 - p)
    finite_set_sum(binomial_support(Nat.1), v) = p * (Real.1 - p)
}

/// The expectation of a Binomial(1, p) PMF with the canonical support and mass
/// is `p`.
theorem binomial_pmf_expectation_one(p: Real, pmf: DiscretePMF[Nat]) {
    Real.0 <= p and p <= Real.1 and pmf.support = binomial_support(Nat.1) and
        pmf.mass = binomial_mass(Nat.1, p)
        implies discrete_expectation(pmf, nat_to_real) = p
} by {
    if Real.0 <= p and p <= Real.1 and pmf.support = binomial_support(Nat.1) and
        pmf.mass = binomial_mass(Nat.1, p) {
        forall(x: Nat) {
            pmf.mass(x) = binomial_mass(Nat.1, p, x)
        }
        forall(x: Nat) {
            mass_weighted_value(pmf.mass, nat_to_real, x) =
                mass_weighted_value(binomial_mass(Nat.1, p), nat_to_real, x)
        }
        function_extensionality[Nat, Real](mass_weighted_value(pmf.mass, nat_to_real),
            mass_weighted_value(binomial_mass(Nat.1, p), nat_to_real))
        mass_weighted_value(pmf.mass, nat_to_real) =
            mass_weighted_value(binomial_mass(Nat.1, p), nat_to_real)
        finite_set_sum(pmf.support, mass_weighted_value(pmf.mass, nat_to_real)) =
            finite_set_sum(binomial_support(Nat.1), mass_weighted_value(binomial_mass(Nat.1, p), nat_to_real))
        binomial_expectation_one(p)
        finite_set_sum(binomial_support(Nat.1),
            mass_weighted_value(binomial_mass(Nat.1, p), nat_to_real)) = p
        discrete_expectation(pmf, nat_to_real) =
            finite_set_sum(pmf.support, mass_weighted_value(pmf.mass, nat_to_real))
        discrete_expectation(pmf, nat_to_real) = p
    }
}

/// The variance of a Binomial(1, p) PMF with the canonical support and mass
/// is `p (1 - p)`.
theorem binomial_pmf_variance_one(p: Real, pmf: DiscretePMF[Nat]) {
    Real.0 <= p and p <= Real.1 and pmf.support = binomial_support(Nat.1) and
        pmf.mass = binomial_mass(Nat.1, p)
        implies discrete_variance(pmf, nat_to_real) = p * (Real.1 - p)
} by {
    if Real.0 <= p and p <= Real.1 and pmf.support = binomial_support(Nat.1) and
        pmf.mass = binomial_mass(Nat.1, p) {
        binomial_pmf_expectation_one(p, pmf)
        discrete_expectation(pmf, nat_to_real) = p
        forall(x: Nat) {
            square_dev(nat_to_real, discrete_expectation(pmf, nat_to_real), x) =
                (nat_to_real(x) - discrete_expectation(pmf, nat_to_real)) * (nat_to_real(x) - discrete_expectation(pmf, nat_to_real))
            square_dev(nat_to_real, p, x) =
                (nat_to_real(x) - p) * (nat_to_real(x) - p)
            square_dev(nat_to_real, discrete_expectation(pmf, nat_to_real), x) =
                square_dev(nat_to_real, p, x)
        }
        function_extensionality[Nat, Real](square_dev(nat_to_real, discrete_expectation(pmf, nat_to_real)),
            square_dev(nat_to_real, p))
        square_dev(nat_to_real, discrete_expectation(pmf, nat_to_real)) = square_dev(nat_to_real, p)
        discrete_expectation(pmf, square_dev(nat_to_real, discrete_expectation(pmf, nat_to_real))) =
            discrete_expectation(pmf, square_dev(nat_to_real, p))
        forall(x: Nat) {
            mass_weighted_value(pmf.mass, square_dev(nat_to_real, p), x) =
                mass_weighted_value(binomial_mass(Nat.1, p), square_dev(nat_to_real, p), x)
        }
        function_extensionality[Nat, Real](mass_weighted_value(pmf.mass, square_dev(nat_to_real, p)),
            mass_weighted_value(binomial_mass(Nat.1, p), square_dev(nat_to_real, p)))
        mass_weighted_value(pmf.mass, square_dev(nat_to_real, p)) =
            mass_weighted_value(binomial_mass(Nat.1, p), square_dev(nat_to_real, p))
        finite_set_sum(pmf.support, mass_weighted_value(pmf.mass, square_dev(nat_to_real, p))) =
            finite_set_sum(binomial_support(Nat.1), mass_weighted_value(binomial_mass(Nat.1, p), square_dev(nat_to_real, p)))
        binomial_variance_one(p)
        finite_set_sum(binomial_support(Nat.1),
            mass_weighted_value(binomial_mass(Nat.1, p), square_dev(nat_to_real, p))) = p * (Real.1 - p)
        discrete_expectation(pmf, square_dev(nat_to_real, p)) =
            finite_set_sum(pmf.support, mass_weighted_value(pmf.mass, square_dev(nat_to_real, p)))
        discrete_expectation(pmf, square_dev(nat_to_real, p)) = p * (Real.1 - p)
        discrete_variance(pmf, nat_to_real) =
            discrete_expectation(pmf, square_dev(nat_to_real, discrete_expectation(pmf, nat_to_real)))
        discrete_variance(pmf, nat_to_real) = p * (Real.1 - p)
    }
}

/// The expectation of the Binomial(2, p) count is `2p`:
/// Σ_{k=0}^{2} k·C(2,k) p^k (1-p)^(2-k) = 2p.
theorem binomial_expectation_two(p: Real) {
    finite_set_sum(binomial_support(Nat.2),
        mass_weighted_value(binomial_mass(Nat.2, p), nat_to_real)) = (Real.1 + Real.1) * p
} by {
    let g = mass_weighted_value(binomial_mass(Nat.2, p), nat_to_real)
    range_sum_eq_finite_set_sum(g, Nat.3)
    finite_set_sum(binomial_support(Nat.2), g) = range_sum(g, Nat.3)
    range_sum_suc(g, Nat.2)
    range_sum(g, Nat.3) = range_sum(g, Nat.2) + g(Nat.2)
    range_sum_suc(g, Nat.1)
    range_sum(g, Nat.2) = range_sum(g, Nat.1) + g(Nat.1)
    range_sum_suc(g, Nat.0)
    range_sum(g, Nat.1) = range_sum(g, Nat.0) + g(Nat.0)
    range_sum_zero(g)
    range_sum(g, Nat.0) = Real.0
    range_sum(g, Nat.3) = Real.0 + g(Nat.0) + g(Nat.1) + g(Nat.2)
    g(Nat.0) = binomial_mass(Nat.2, p, Nat.0) * nat_to_real(Nat.0)
    nat_to_real(Nat.0) = from_nat[Real](Nat.0)
    from_nat[Real](Nat.0) = Real.0
    g(Nat.0) = binomial_mass(Nat.2, p, Nat.0) * Real.0
    binomial_mass(Nat.2, p, Nat.0) * Real.0 = Real.0
    g(Nat.0) = Real.0
    g(Nat.1) = binomial_mass(Nat.2, p, Nat.1) * nat_to_real(Nat.1)
    nat_to_real(Nat.1) = from_nat[Real](Nat.1)
    from_nat[Real](Nat.1) = Real.1
    g(Nat.1) = binomial_mass(Nat.2, p, Nat.1) * Real.1
    binomial_mass_two_one(p)
    binomial_mass(Nat.2, p, Nat.1) = (Real.1 + Real.1) * p * (Real.1 - p)
    g(Nat.1) = (Real.1 + Real.1) * p * (Real.1 - p) * Real.1
    (Real.1 + Real.1) * p * (Real.1 - p) * Real.1 = (Real.1 + Real.1) * p * (Real.1 - p)
    g(Nat.1) = (Real.1 + Real.1) * p * (Real.1 - p)
    g(Nat.2) = binomial_mass(Nat.2, p, Nat.2) * nat_to_real(Nat.2)
    nat_to_real(Nat.2) = from_nat[Real](Nat.2)
    from_nat_two_real
    from_nat[Real](Nat.2) = Real.1 + Real.1
    nat_to_real(Nat.2) = Real.1 + Real.1
    binomial_mass_two_two(p)
    binomial_mass(Nat.2, p, Nat.2) = p * p
    g(Nat.2) = (p * p) * (Real.1 + Real.1)
    (p * p) * (Real.1 + Real.1) = (Real.1 + Real.1) * (p * p)
    g(Nat.2) = (Real.1 + Real.1) * (p * p)
    (Real.1 + Real.1) * p * (Real.1 - p) + (Real.1 + Real.1) * (p * p) = (Real.1 + Real.1) * (p * (Real.1 - p) + p * p)
    p * (Real.1 - p) + p * p = p * ((Real.1 - p) + p)
    real_p_add_one_sub(p)
    p + (Real.1 - p) = Real.1
    (Real.1 - p) + p = Real.1
    p * ((Real.1 - p) + p) = p * Real.1
    p * Real.1 = p
    (Real.1 + Real.1) * (p * (Real.1 - p) + p * p) = (Real.1 + Real.1) * p
    Real.0 + Real.0 + (Real.1 + Real.1) * p * (Real.1 - p) + (Real.1 + Real.1) * (p * p) = (Real.1 + Real.1) * p
    finite_set_sum(binomial_support(Nat.2), g) = (Real.1 + Real.1) * p
}

/// The expectation of a Binomial(2, p) PMF with the canonical support and mass
/// is `2p`.
theorem binomial_pmf_expectation_two(p: Real, pmf: DiscretePMF[Nat]) {
    Real.0 <= p and p <= Real.1 and pmf.support = binomial_support(Nat.2) and
        pmf.mass = binomial_mass(Nat.2, p)
        implies discrete_expectation(pmf, nat_to_real) = (Real.1 + Real.1) * p
} by {
    if Real.0 <= p and p <= Real.1 and pmf.support = binomial_support(Nat.2) and
        pmf.mass = binomial_mass(Nat.2, p) {
        forall(x: Nat) {
            pmf.mass(x) = binomial_mass(Nat.2, p, x)
        }
        forall(x: Nat) {
            mass_weighted_value(pmf.mass, nat_to_real, x) =
                mass_weighted_value(binomial_mass(Nat.2, p), nat_to_real, x)
        }
        function_extensionality[Nat, Real](mass_weighted_value(pmf.mass, nat_to_real),
            mass_weighted_value(binomial_mass(Nat.2, p), nat_to_real))
        mass_weighted_value(pmf.mass, nat_to_real) =
            mass_weighted_value(binomial_mass(Nat.2, p), nat_to_real)
        finite_set_sum(pmf.support, mass_weighted_value(pmf.mass, nat_to_real)) =
            finite_set_sum(binomial_support(Nat.2), mass_weighted_value(binomial_mass(Nat.2, p), nat_to_real))
        binomial_expectation_two(p)
        finite_set_sum(binomial_support(Nat.2),
            mass_weighted_value(binomial_mass(Nat.2, p), nat_to_real)) = (Real.1 + Real.1) * p
        discrete_expectation(pmf, nat_to_real) =
            finite_set_sum(pmf.support, mass_weighted_value(pmf.mass, nat_to_real))
        discrete_expectation(pmf, nat_to_real) = (Real.1 + Real.1) * p
    }
}

// The general expectation of the Binomial(n, p) count is n p:
//   E[X] = Σ_{k=0}^{n} k·C(n,k) p^k (1-p)^(n-k) = n p,
// by the absorption identity k·C(n,k) = n·C(n-1,k-1) and the binomial theorem.
// (Stated for the verified instance n = 2 above.)

// The general variance of the Binomial(n, p) count is n p (1 - p):
//   Var(X) = E[X^2] - E[X]^2 = n p (1 - p),
// via the absorption identity and the binomial theorem.
// (Stated for the verified instance n = 1 above.)

// The mode of the Binomial(n, p) distribution: the most likely value of X is
// the largest k with (n+1)p - 1 <= k <= (n+1)p, approximately n p; the ratio of
// consecutive masses P(X = k+1)/P(X = k) = (n - k)p / ((k+1)(1 - p)) is
// decreasing in k, so the mode is the integer nearest (n+1)p.
