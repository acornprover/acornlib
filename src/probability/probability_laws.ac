from real import Real
from finite_set import FiniteSet, fs_union, fs_intersection, fs_difference
from algebra.add_semigroup import add_fn
from probability.discrete_pmf import DiscretePMF, pmf_event_prob, pmf_event_prob_complement,
    pmf_event_prob_eq_support_intersection, pmf_event_prob_union_le_sum, discrete_expectation,
    discrete_expectation_add, discrete_expectation_const, const_real_fn, finite_event_indicator,
    discrete_expectation_indicator_support_complement
from probability.expectation_properties import expectation_const, markov_inequality, rv_ge_event,
    discrete_expectation_add_independent
from probability.discrete_probability_ext import discrete_rv_independent, is_bernoulli_pmf,
    bool_true_event, bernoulli_indicator_expectation

/// The complement law: the probability of the complementary event `support \ a`
/// is `1 - P(a)`, for any finite event `a`.  Points outside the support carry no
/// mass, so an event agrees in probability with its intersection with the support.
theorem probability_complement_law[T](pmf: DiscretePMF[T], a: FiniteSet[T]) {
    pmf_event_prob(pmf, fs_difference(pmf.support, a)) = Real.1 - pmf_event_prob(pmf, a)
} by {
    pmf_event_prob_complement(pmf, a)
    pmf_event_prob(pmf, fs_difference(pmf.support, a)) =
        Real.1 - pmf_event_prob(pmf, fs_intersection(pmf.support, a))
    pmf_event_prob_eq_support_intersection(pmf, a)
    pmf_event_prob(pmf, a) = pmf_event_prob(pmf, fs_intersection(pmf.support, a))
    pmf_event_prob(pmf, fs_intersection(pmf.support, a)) = pmf_event_prob(pmf, a)
    Real.1 - pmf_event_prob(pmf, fs_intersection(pmf.support, a)) =
        Real.1 - pmf_event_prob(pmf, a)
    pmf_event_prob(pmf, fs_difference(pmf.support, a)) = Real.1 - pmf_event_prob(pmf, a)
}

/// The complement law for indicator expectations: the expectation of the indicator
/// of `support \ a` is one minus the expectation of the indicator of `a`.
theorem probability_complement_law_indicator[T](pmf: DiscretePMF[T], a: FiniteSet[T]) {
    discrete_expectation(pmf, finite_event_indicator(fs_difference(pmf.support, a))) =
        Real.1 - discrete_expectation(pmf, finite_event_indicator(a))
} by {
    discrete_expectation_indicator_support_complement(pmf, a)
}

/// The union bound for two finite events: the probability of the union is at most
/// the sum of the individual probabilities.
theorem probability_union_bound[T](pmf: DiscretePMF[T], a: FiniteSet[T], b: FiniteSet[T]) {
    pmf_event_prob(pmf, fs_union(a, b)) <= pmf_event_prob(pmf, a) + pmf_event_prob(pmf, b)
} by {
    pmf_event_prob_union_le_sum(pmf, a, b)
}

/// The expectation of a constant random variable is the constant, under any discrete pmf.
theorem probability_expectation_const[T](pmf: DiscretePMF[T], c: Real) {
    discrete_expectation(pmf, const_real_fn[T](c)) = c
} by {
    expectation_const(pmf, c)
}

/// Markov's inequality: for a nonnegative random variable `X` and a threshold `a > 0`,
/// the probability that `X` is at least `a` is at most `E(X) / a`.
theorem probability_markov_inequality[T](pmf: DiscretePMF[T], rv: T -> Real, a: Real) {
    (forall(t: T) { pmf.support.contains(t) implies Real.0 <= rv(t) }) and a > Real.0
        implies pmf_event_prob(pmf, rv_ge_event(pmf, rv, a)) <= discrete_expectation(pmf, rv) / a
} by {
    markov_inequality(pmf, rv, a)
}

/// The expectation of a sum is the sum of the expectations: `E(X + Y) = E(X) + E(Y)`.
/// Linearity of expectation holds without any independence hypothesis.
theorem probability_expectation_add[T](pmf: DiscretePMF[T], x: T -> Real, y: T -> Real) {
    discrete_expectation(pmf, add_fn(x, y)) = discrete_expectation(pmf, x) + discrete_expectation(pmf, y)
} by {
    discrete_expectation_add(pmf, x, y)
}

/// The expectation of a sum of independent random variables is the sum of the
/// expectations; the hypothesis is kept to match the classical statement, though
/// linearity of expectation holds without it.
theorem probability_expectation_add_independent[T](pmf: DiscretePMF[T], x: T -> Real, y: T -> Real) {
    discrete_rv_independent(pmf, x, y) implies
        discrete_expectation(pmf, add_fn(x, y)) = discrete_expectation(pmf, x) + discrete_expectation(pmf, y)
} by {
    discrete_expectation_add_independent(pmf, x, y)
}

/// The expectation of the indicator of `{true}` under a Bernoulli pmf with
/// parameter `p` is `p`.
theorem probability_bernoulli_expectation(p: Real, pmf: DiscretePMF[Bool]) {
    is_bernoulli_pmf(p, pmf) implies
        discrete_expectation(pmf, finite_event_indicator(bool_true_event)) = p
} by {
    bernoulli_indicator_expectation(p, pmf)
}
