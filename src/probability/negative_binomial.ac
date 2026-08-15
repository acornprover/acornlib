/// The negative binomial distribution: the law of the number of failures
/// before the r-th success in a sequence of independent Bernoulli trials with
/// success probability `p`.

from nat import Nat, from_nat, from_nat_zero, from_nat_one, alt_induction
from real import Real, mul_nonneg
from order import lt_imp_lte
from ordered_field import zero_is_smaller_than_one
from combinatorics import binom
from probability.bernoulli_pmf import one_sub_nonneg
from probability.binomial import from_nat_nonneg_real, real_pow_nonneg_of_nonneg, gte_zero_unfold

numerals Nat

/// The negative binomial probability mass with `r` successes and parameter `p`
/// at failure count `j`:
/// P(X = r + j) = C(r+j-1, r-1) p^r (1-p)^j,
/// the probability that the r-th success occurs on trial r + j.  The special
/// case r = 1 is the geometric distribution.
define negative_binomial_mass(r: Nat, p: Real, j: Nat) -> Real {
    from_nat[Real]((r + j - 1).binom(r - 1)) * p.pow(r) * (Real.1 - p).pow(j)
}

/// The negative binomial mass is nonnegative when `0 <= p <= 1`.
theorem negative_binomial_mass_nonneg(r: Nat, p: Real, j: Nat) {
    Real.0 <= p and p <= Real.1 implies Real.0 <= negative_binomial_mass(r, p, j)
} by {
    if Real.0 <= p and p <= Real.1 {
        from_nat_nonneg_real((r + j - 1).binom(r - 1))
        Real.0 <= from_nat[Real]((r + j - 1).binom(r - 1))
        real_pow_nonneg_of_nonneg(p, r)
        Real.0 <= p.pow(r)
        one_sub_nonneg(p)
        Real.0 <= Real.1 - p
        real_pow_nonneg_of_nonneg(Real.1 - p, j)
        Real.0 <= (Real.1 - p).pow(j)
        gte_zero_unfold(from_nat[Real]((r + j - 1).binom(r - 1)))
        from_nat[Real]((r + j - 1).binom(r - 1)) >= Real.0
        gte_zero_unfold(p.pow(r))
        p.pow(r) >= Real.0
        mul_nonneg(from_nat[Real]((r + j - 1).binom(r - 1)), p.pow(r))
        from_nat[Real]((r + j - 1).binom(r - 1)) * p.pow(r) >= Real.0
        gte_zero_unfold(from_nat[Real]((r + j - 1).binom(r - 1)) * p.pow(r))
        Real.0 <= from_nat[Real]((r + j - 1).binom(r - 1)) * p.pow(r)
        gte_zero_unfold((Real.1 - p).pow(j))
        (Real.1 - p).pow(j) >= Real.0
        mul_nonneg(from_nat[Real]((r + j - 1).binom(r - 1)) * p.pow(r), (Real.1 - p).pow(j))
        from_nat[Real]((r + j - 1).binom(r - 1)) * p.pow(r) * (Real.1 - p).pow(j) >= Real.0
        gte_zero_unfold(from_nat[Real]((r + j - 1).binom(r - 1)) * p.pow(r) * (Real.1 - p).pow(j))
        Real.0 <= from_nat[Real]((r + j - 1).binom(r - 1)) * p.pow(r) * (Real.1 - p).pow(j)
        negative_binomial_mass(r, p, j) =
            from_nat[Real]((r + j - 1).binom(r - 1)) * p.pow(r) * (Real.1 - p).pow(j)
        Real.0 <= negative_binomial_mass(r, p, j)
    }
}

/// The r = 1 negative binomial mass is the geometric mass: the first success
/// occurs after `j` failures with probability p (1-p)^j.
theorem negative_binomial_mass_one(p: Real, j: Nat) {
    negative_binomial_mass(Nat.1, p, j) = p * (Real.1 - p).pow(j)
} by {
    negative_binomial_mass(Nat.1, p, j) =
        from_nat[Real]((Nat.1 + j - 1).binom(Nat.1 - 1)) * p.pow(Nat.1) * (Real.1 - p).pow(j)
    Nat.1 - 1 = Nat.0
    Nat.1 + j - 1 = j
    (Nat.1 + j - 1).binom(Nat.1 - 1) = j.binom(Nat.0)
    j.binom(Nat.0) = Nat.1
    from_nat_one[Real]
    from_nat[Real](Nat.1) = Real.1
    from_nat[Real]((Nat.1 + j - 1).binom(Nat.1 - 1)) = Real.1
    p.pow(Nat.1) = p
    Real.1 * p * (Real.1 - p).pow(j) = p * (Real.1 - p).pow(j)
    negative_binomial_mass(Nat.1, p, j) = p * (Real.1 - p).pow(j)
}

/// The negative binomial probabilities sum to one:
///   Σ_{j>=0} C(r+j-1, r-1) p^r (1-p)^j = 1.
///
/// The series route mirrors the geometric law: with q = 1-p and
/// S_r = Σ_j C(r+j-1, r-1) q^j, Pascal's identity and an index shift give the
/// recurrence partial(n+1) = q partial(n) + partial(r-series, n+1) for the
/// coefficient partial sums, so the limits obey S_{r+1} (1-q) = S_r.  With
/// S_1 = 1/(1-q) this yields S_r = (1-q)^{-r}, hence
/// p^r S_r = (p/(1-q))^r = 1.  The verification of this series induction is
/// left as a statement here, in the style of the general binomial moments in
/// binomial.ac.
///
/// The probability that the r-th success occurs on trial `k` (k >= r) is
/// negative_binomial_mass(r, p, k - r).
