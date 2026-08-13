from real import Real
from finite_set import FiniteSet
from data.finite.finite_set_filter import finite_set_filter, finite_set_filter_contains_eq
from data.basic.functions import function_extensionality
from probability.discrete_pmf import DiscretePMF, finite_event_indicator, discrete_expectation,
    discrete_expectation_indicator_eq_event_prob, pmf_event_prob
from probability.discrete_probability_ext import rv_value_event

/// The expectation of an event indicator is the probability of the event:
/// `E(1_A) = P(A)` for every finite event `A`.  This restates
/// `discrete_expectation_indicator_eq_event_prob` from `discrete_pmf` under a
/// name that matches the classical statement.
theorem expectation_indicator_eq_prob[T](pmf: DiscretePMF[T], event: FiniteSet[T]) {
    discrete_expectation(pmf, finite_event_indicator(event)) = pmf_event_prob(pmf, event)
} by {
    discrete_expectation_indicator_eq_event_prob(pmf, event)
}

/// The expectation of the indicator of the value event `{X = a}` is the
/// probability that `X` takes the value `a`.
theorem expectation_indicator_value_event_eq_prob[T](pmf: DiscretePMF[T], rv: T -> Real, a: Real) {
    discrete_expectation(pmf, finite_event_indicator(rv_value_event(pmf, rv, a))) =
        pmf_event_prob(pmf, rv_value_event(pmf, rv, a))
} by {
    discrete_expectation_indicator_eq_event_prob(pmf, rv_value_event(pmf, rv, a))
}

/// True when the value `rv(x)` of a random variable lies in a finite set of values.
define rv_in_event_contains[T, A](rv: T -> A, event: FiniteSet[A], x: T) -> Bool {
    event.contains(rv(x))
}

/// The preimage event of a finite value set: the elements of the support whose
/// value under `rv` lies in `event`.  Events are finite sets in this framework,
/// so the preimage is taken inside the (finite) support.
define rv_in_event[T, A](pmf: DiscretePMF[T], rv: T -> A, event: FiniteSet[A]) -> FiniteSet[T] {
    finite_set_filter(pmf.support, rv_in_event_contains(rv, event))
}

/// Membership in a preimage event is support membership together with value membership.
theorem rv_in_event_contains_eq[T, A](pmf: DiscretePMF[T], rv: T -> A, event: FiniteSet[A], x: T) {
    rv_in_event(pmf, rv, event).contains(x) = (pmf.support.contains(x) and event.contains(rv(x)))
} by {
    rv_in_event_contains(rv, event, x) = event.contains(rv(x))
    finite_set_filter_contains_eq(pmf.support, rv_in_event_contains(rv, event), x)
    finite_set_filter(pmf.support, rv_in_event_contains(rv, event)).contains(x) =
        (pmf.support.contains(x) and rv_in_event_contains(rv, event, x))
    finite_set_filter(pmf.support, rv_in_event_contains(rv, event)).contains(x) =
        (pmf.support.contains(x) and event.contains(rv(x)))
    rv_in_event(pmf, rv, event) = finite_set_filter(pmf.support, rv_in_event_contains(rv, event))
    rv_in_event(pmf, rv, event).contains(x) = (pmf.support.contains(x) and event.contains(rv(x)))
}

/// The indicator of the preimage event `{X in S}`: one at outcomes where the
/// random variable takes a value in the finite set `S`, zero elsewhere.
define rv_in_event_indicator[T, A](pmf: DiscretePMF[T], rv: T -> A, event: FiniteSet[A], x: T) -> Real {
    finite_event_indicator(rv_in_event(pmf, rv, event), x)
}

/// The indicator of a preimage event equals the indicator of the event itself.
theorem rv_in_event_indicator_eq_indicator[T, A](pmf: DiscretePMF[T], rv: T -> A, event: FiniteSet[A]) {
    rv_in_event_indicator(pmf, rv, event) = finite_event_indicator(rv_in_event(pmf, rv, event))
} by {
    forall(x: T) {
        rv_in_event_indicator(pmf, rv, event, x) =
            finite_event_indicator(rv_in_event(pmf, rv, event), x)
    }
    function_extensionality[T, Real](rv_in_event_indicator(pmf, rv, event),
        finite_event_indicator(rv_in_event(pmf, rv, event)))
}

/// The expectation of the indicator of the preimage event `{X in S}` is the
/// probability that `X` takes a value in `S`: `E(1_{X in S}) = P(X in S)`.
theorem expectation_indicator_in_event_eq_prob[T, A](pmf: DiscretePMF[T], rv: T -> A, event: FiniteSet[A]) {
    discrete_expectation(pmf, rv_in_event_indicator(pmf, rv, event)) =
        pmf_event_prob(pmf, rv_in_event(pmf, rv, event))
} by {
    rv_in_event_indicator_eq_indicator(pmf, rv, event)
    rv_in_event_indicator(pmf, rv, event) = finite_event_indicator(rv_in_event(pmf, rv, event))
    discrete_expectation(pmf, rv_in_event_indicator(pmf, rv, event)) =
        discrete_expectation(pmf, finite_event_indicator(rv_in_event(pmf, rv, event)))
    discrete_expectation_indicator_eq_event_prob(pmf, rv_in_event(pmf, rv, event))
    discrete_expectation(pmf, finite_event_indicator(rv_in_event(pmf, rv, event))) =
        pmf_event_prob(pmf, rv_in_event(pmf, rv, event))
    discrete_expectation(pmf, rv_in_event_indicator(pmf, rv, event)) =
        pmf_event_prob(pmf, rv_in_event(pmf, rv, event))
}
