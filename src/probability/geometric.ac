/// The geometric distribution: the law of the number of failures before the
/// first success in a sequence of independent Bernoulli trials with success
/// probability `p`.
///
/// The geometric mass function assigns to each natural number `k` the
/// probability P(X = k) = p (1-p)^k.  The outcome space `Nat` is infinite, so
/// like the Poisson law (see `poisson.ac`) the distribution is formalized here
/// at the level of the mass function and the convergence of its series rather
/// than as a bundled `DiscretePMF`.

from nat import Nat, from_nat, from_nat_zero, from_nat_one, from_nat_add, from_nat_mul,
    pow_zero, pow_one, pow_add, one_pow, alt_induction
from real import Real, mul_nonneg, converges, converges_to, limit, mul_seq, mul_seq_converges_to,
    tail, tail_imp_converges_to, seq_pointwise_eq_converges_to,
    converges_to_imp_converges, converges_to_unique, convergent_converges_to_limit,
    limit_add_seq, add_seq, geom_converges, geom_series, is_increasing, is_upper_bound,
    monotone_convergence_principle, increasing_convergent_bounded_by_limit,
    zero_lte_imp_non_neg, abs_of_nonneg, mul_left_cancel, mul_div_cancel
from order import lt_imp_lte
from ordered_field import zero_is_smaller_than_one, mul_le_mul_of_nonneg_left,
    positive_pows_of_positive_are_positive
from algebra.semigroup import mul_fn
from algebra.add_semigroup import add_fn
from algebra.add_ordered_group import add_le_add, add_lt_add_left, add_lt_add_right
from data.basic.functions import function_extensionality, compose
from list import partial, partial_add, partial_scalar_mul, partial_shift_suc, partial_pointwise_eq,
    partial_split_last, partial_zero, partial_one
from probability.bernoulli_pmf import one_sub_nonneg
from probability.binomial import from_nat_nonneg_real, real_pow_nonneg_of_nonneg, gte_zero_unfold
from real import square_nonneg, lte_trans

/// The probability mass of the geometric distribution with parameter `p` at `k`:
/// P(X = k) = p (1-p)^k for k = 0, 1, 2, ...; the number of failures before the
/// first success in independent trials with success probability `p`.
define geometric_mass(p: Real, k: Nat) -> Real {
    p * (Real.1 - p).pow(k)
}

/// The k-th summand of the geometric expectation: k p (1-p)^k.
define geometric_mean_term(p: Real, k: Nat) -> Real {
    from_nat[Real](k) * geometric_mass(p, k)
}

/// The k-th summand of the geometric second moment: k^2 p (1-p)^k.
define geometric_second_moment_term(p: Real, k: Nat) -> Real {
    from_nat[Real](k) * from_nat[Real](k) * geometric_mass(p, k)
}

/// The k-th term of the weighted geometric series: k q^k.
define weighted_geom_term(q: Real, k: Nat) -> Real {
    from_nat[Real](k) * q.pow(k)
}

/// The geometric mass is nonnegative when `0 <= p <= 1`.
theorem geometric_mass_nonneg(p: Real, k: Nat) {
    Real.0 <= p and p <= Real.1 implies Real.0 <= geometric_mass(p, k)
} by {
    if Real.0 <= p and p <= Real.1 {
        one_sub_nonneg(p)
        Real.0 <= Real.1 - p
        real_pow_nonneg_of_nonneg(Real.1 - p, k)
        Real.0 <= (Real.1 - p).pow(k)
        gte_zero_unfold(p)
        p >= Real.0
        gte_zero_unfold((Real.1 - p).pow(k))
        (Real.1 - p).pow(k) >= Real.0
        mul_nonneg(p, (Real.1 - p).pow(k))
        p * (Real.1 - p).pow(k) >= Real.0
        gte_zero_unfold(p * (Real.1 - p).pow(k))
        Real.0 <= p * (Real.1 - p).pow(k)
        geometric_mass(p, k) = p * (Real.1 - p).pow(k)
        Real.0 <= geometric_mass(p, k)
    }
}

/// The weighted geometric term `k q^k` is nonnegative when `0 <= q`.
theorem weighted_geom_term_nonneg(q: Real, k: Nat) {
    Real.0 <= q implies Real.0 <= weighted_geom_term(q, k)
} by {
    if Real.0 <= q {
        from_nat_nonneg_real(k)
        Real.0 <= from_nat[Real](k)
        real_pow_nonneg_of_nonneg(q, k)
        Real.0 <= q.pow(k)
        gte_zero_unfold(from_nat[Real](k))
        from_nat[Real](k) >= Real.0
        gte_zero_unfold(q.pow(k))
        q.pow(k) >= Real.0
        mul_nonneg(from_nat[Real](k), q.pow(k))
        from_nat[Real](k) * q.pow(k) >= Real.0
        gte_zero_unfold(from_nat[Real](k) * q.pow(k))
        Real.0 <= from_nat[Real](k) * q.pow(k)
        weighted_geom_term(q, k) = from_nat[Real](k) * q.pow(k)
        Real.0 <= weighted_geom_term(q, k)
    }
}

/// The geometric mass is the pointwise product of `p` and the `(1-p)`-th powers.
lemma geometric_mass_eq_mul_fn(p: Real) {
    geometric_mass(p) = mul_fn(p, (Real.1 - p).pow)
} by {
    forall(k: Nat) {
        geometric_mass(p, k) = p * (Real.1 - p).pow(k)
        mul_fn(p, (Real.1 - p).pow, k) = p * (Real.1 - p).pow(k)
        geometric_mass(p, k) = mul_fn(p, (Real.1 - p).pow, k)
    }
    function_extensionality[Nat, Real](geometric_mass(p), mul_fn(p, (Real.1 - p).pow))
}

/// The ratio `q = 1 - p` of a geometric distribution with `0 < p <= 1` is
/// nonnegative and strictly below one.
theorem geometric_ratio_bounds(p: Real) {
    Real.0 < p and p <= Real.1 implies
        Real.0 <= Real.1 - p and (Real.1 - p).abs < Real.1
} by {
    if Real.0 < p and p <= Real.1 {
        one_sub_nonneg(p)
        Real.0 <= Real.1 - p
        -p < Real.0
        add_lt_add_left[Real](-p, Real.0, Real.1)
        Real.1 + -p < Real.1 + Real.0
        Real.1 - p < Real.1
        gte_zero_unfold(Real.1 - p)
        (Real.1 - p) >= Real.0
        abs_of_nonneg(Real.1 - p)
        (Real.1 - p).abs = Real.1 - p
        (Real.1 - p).abs < Real.1
    }
}

/// The partial sums of the geometric series are the `p`-scaled partial sums of
/// the geometric series in the ratio `1 - p`.
lemma geometric_partial_eq_mul_seq(p: Real, n: Nat) {
    partial(geometric_mass(p), n) = mul_seq(p, partial((Real.1 - p).pow), n)
} by {
    geometric_mass_eq_mul_fn(p)
    geometric_mass(p) = mul_fn(p, (Real.1 - p).pow)
    forall(k: Nat) {
        if k < n {
            geometric_mass(p, k) = mul_fn(p, (Real.1 - p).pow, k)
        }
    }
    partial_pointwise_eq(geometric_mass(p), mul_fn(p, (Real.1 - p).pow), n)
    partial(geometric_mass(p), n) = partial(mul_fn(p, (Real.1 - p).pow), n)
    partial_scalar_mul(p, (Real.1 - p).pow, n)
    p * partial((Real.1 - p).pow, n) = partial(mul_fn(p, (Real.1 - p).pow), n)
    partial(geometric_mass(p), n) = p * partial((Real.1 - p).pow, n)
    mul_seq(p, partial((Real.1 - p).pow), n) = p * partial((Real.1 - p).pow, n)
    partial(geometric_mass(p), n) = mul_seq(p, partial((Real.1 - p).pow), n)
}

/// The geometric series converges, to `p` times the limit of the geometric
/// series in the ratio `1 - p`.
theorem geometric_series_converges_to(p: Real) {
    Real.0 < p and p <= Real.1 implies
        converges_to(partial(geometric_mass(p)), p * limit(partial((Real.1 - p).pow)))
} by {
    if Real.0 < p and p <= Real.1 {
        geometric_ratio_bounds(p)
        (Real.1 - p).abs < Real.1
        geom_converges(Real.1 - p)
        converges(partial((Real.1 - p).pow))
        mul_seq_converges_to(p, partial((Real.1 - p).pow))
        converges_to(mul_seq(p, partial((Real.1 - p).pow)), p * limit(partial((Real.1 - p).pow)))
        forall(n: Nat) {
            geometric_partial_eq_mul_seq(p, n)
            partial(geometric_mass(p), n) = mul_seq(p, partial((Real.1 - p).pow), n)
        }
        seq_pointwise_eq_converges_to(mul_seq(p, partial((Real.1 - p).pow)), partial(geometric_mass(p)), p * limit(partial((Real.1 - p).pow)))
        converges_to(partial(geometric_mass(p)), p * limit(partial((Real.1 - p).pow)))
    }
}

/// The partial sums of the geometric series converge for `0 < p <= 1`.
theorem geometric_series_converges(p: Real) {
    Real.0 < p and p <= Real.1 implies converges(partial(geometric_mass(p)))
} by {
    if Real.0 < p and p <= Real.1 {
        geometric_series_converges_to(p)
        converges_to(partial(geometric_mass(p)), p * limit(partial((Real.1 - p).pow)))
        converges_to_imp_converges(partial(geometric_mass(p)), p * limit(partial((Real.1 - p).pow)))
        converges(partial(geometric_mass(p)))
    }
}

/// The geometric probabilities sum to one: Σ_{k>=0} p (1-p)^k = 1.
theorem geometric_series_sums_to_one(p: Real) {
    Real.0 < p and p <= Real.1 implies limit(partial(geometric_mass(p))) = Real.1
} by {
    if Real.0 < p and p <= Real.1 {
        geometric_series_converges(p)
        converges(partial(geometric_mass(p)))
        convergent_converges_to_limit(partial(geometric_mass(p)))
        converges_to(partial(geometric_mass(p)), limit(partial(geometric_mass(p))))
        geometric_series_converges_to(p)
        converges_to(partial(geometric_mass(p)), p * limit(partial((Real.1 - p).pow)))
        converges_to_unique(partial(geometric_mass(p)), limit(partial(geometric_mass(p))), p * limit(partial((Real.1 - p).pow)))
        limit(partial(geometric_mass(p))) = p * limit(partial((Real.1 - p).pow))

        geometric_ratio_bounds(p)
        (Real.1 - p).abs < Real.1
        geom_series(Real.1 - p)
        limit(partial((Real.1 - p).pow)) = Real.1 / (Real.1 - (Real.1 - p))
        Real.1 - (Real.1 - p) = p
        limit(partial((Real.1 - p).pow)) = Real.1 / p
        limit(partial(geometric_mass(p))) = p * (Real.1 / p)
        p != Real.0
        p * (Real.1 / p) = Real.1
        limit(partial(geometric_mass(p))) = Real.1
    }
}

/// Successor recurrence for the weighted geometric terms:
/// (k+1) q^(k+1) = q (k q^k) + q q^k.
lemma weighted_geom_term_suc(q: Real, k: Nat) {
    weighted_geom_term(q, k.suc) = q * weighted_geom_term(q, k) + q * q.pow(k)
} by {
    weighted_geom_term(q, k.suc) = from_nat[Real](k.suc) * q.pow(k.suc)
    from_nat_add[Real](k, Nat.1)
    from_nat[Real](k.suc) = from_nat[Real](k) + from_nat[Real](Nat.1)
    from_nat_one[Real]
    from_nat[Real](Nat.1) = Real.1
    from_nat[Real](k.suc) = from_nat[Real](k) + Real.1
    q.pow(k.suc) = q * q.pow(k)
    from_nat[Real](k.suc) * q.pow(k.suc) = (from_nat[Real](k) + Real.1) * (q * q.pow(k))
    weighted_geom_term(q, k) = from_nat[Real](k) * q.pow(k)
    (from_nat[Real](k) + Real.1) * (q * q.pow(k)) = q * (from_nat[Real](k) * q.pow(k)) + q * q.pow(k)
    weighted_geom_term(q, k.suc) = q * weighted_geom_term(q, k) + q * q.pow(k)
}

/// The partial sums of the weighted geometric series satisfy
/// partial(n+1) = q partial(n) + q partial(q.pow, n).
lemma weighted_geom_partial_suc(q: Real, n: Nat) {
    partial(weighted_geom_term(q), n.suc) =
        q * partial(weighted_geom_term(q), n) + q * partial(q.pow, n)
} by {
    weighted_geom_term(q, Nat.0) = from_nat[Real](Nat.0) * q.pow(Nat.0)
    from_nat_zero[Real]
    from_nat[Real](Nat.0) = Real.0
    Real.0 * q.pow(Nat.0) = Real.0
    weighted_geom_term(q, Nat.0) = Real.0

    partial_shift_suc(weighted_geom_term(q), n)
    weighted_geom_term(q, Nat.0) + partial(compose(weighted_geom_term(q), Nat.suc), n) =
        partial(weighted_geom_term(q), n.suc)
    partial(compose(weighted_geom_term(q), Nat.suc), n) = partial(weighted_geom_term(q), n.suc)

    forall(k: Nat) {
        weighted_geom_term_suc(q, k)
        weighted_geom_term(q, k.suc) = q * weighted_geom_term(q, k) + q * q.pow(k)
        compose(weighted_geom_term(q), Nat.suc, k) = weighted_geom_term(q, Nat.suc(k))
        Nat.suc(k) = k.suc
        compose(weighted_geom_term(q), Nat.suc, k) = weighted_geom_term(q, k.suc)
        add_fn(mul_fn(q, weighted_geom_term(q)), mul_fn(q, q.pow), k) =
            mul_fn(q, weighted_geom_term(q), k) + mul_fn(q, q.pow, k)
        mul_fn(q, weighted_geom_term(q), k) = q * weighted_geom_term(q, k)
        mul_fn(q, q.pow, k) = q * q.pow(k)
        compose(weighted_geom_term(q), Nat.suc, k) =
            add_fn(mul_fn(q, weighted_geom_term(q)), mul_fn(q, q.pow), k)
    }
    function_extensionality[Nat, Real](compose(weighted_geom_term(q), Nat.suc),
        add_fn(mul_fn(q, weighted_geom_term(q)), mul_fn(q, q.pow)))
    compose(weighted_geom_term(q), Nat.suc) =
        add_fn(mul_fn(q, weighted_geom_term(q)), mul_fn(q, q.pow))

    partial_add(mul_fn(q, weighted_geom_term(q)), mul_fn(q, q.pow), n)
    partial(add_fn(mul_fn(q, weighted_geom_term(q)), mul_fn(q, q.pow)), n) =
        partial(mul_fn(q, weighted_geom_term(q)), n) + partial(mul_fn(q, q.pow), n)
    partial_scalar_mul(q, weighted_geom_term(q), n)
    q * partial(weighted_geom_term(q), n) = partial(mul_fn(q, weighted_geom_term(q)), n)
    partial_scalar_mul(q, q.pow, n)
    q * partial(q.pow, n) = partial(mul_fn(q, q.pow), n)
    partial(compose(weighted_geom_term(q), Nat.suc), n) =
        q * partial(weighted_geom_term(q), n) + q * partial(q.pow, n)
    partial(weighted_geom_term(q), n.suc) =
        q * partial(weighted_geom_term(q), n) + q * partial(q.pow, n)
}

/// The partial sums of a sequence of nonnegative terms are increasing.
lemma partial_increasing_of_nonneg(f: Nat -> Real) {
    (forall(k: Nat) { Real.0 <= f(k) }) implies is_increasing(partial(f))
} by {
    if forall(k: Nat) { Real.0 <= f(k) } {
        forall(n: Nat) {
            partial_split_last(f, n)
            partial(f, n.suc) = partial(f, n) + f(n)
            Real.0 <= f(n)
            add_le_add[Real](Real.0, f(n), partial(f, n), partial(f, n))
            Real.0 + partial(f, n) <= f(n) + partial(f, n)
            Real.0 + partial(f, n) = partial(f, n)
            f(n) + partial(f, n) = partial(f, n) + f(n)
            partial(f, n) <= partial(f, n.suc)
        }
        forall(n: Nat) {
            partial(f, n) <= partial(f, n.suc)
        }
        is_increasing(partial(f)) = forall(n: Nat) { partial(f, n) <= partial(f, n.suc) }
        is_increasing(partial(f))
    }
}

/// The ordinary geometric series in a ratio `0 <= q < 1` is bounded by its limit.
lemma geom_partial_bounded_by_limit(q: Real) {
    Real.0 <= q and q < Real.1 implies
        (forall(n: Nat) { partial(q.pow, n) <= limit(partial(q.pow)) })
} by {
    if Real.0 <= q and q < Real.1 {
        forall(k: Nat) {
            real_pow_nonneg_of_nonneg(q, k)
            Real.0 <= q.pow(k)
        }
        partial_increasing_of_nonneg(q.pow)
        is_increasing(partial(q.pow))
        gte_zero_unfold(q)
        q >= Real.0
        abs_of_nonneg(q)
        q.abs = q
        q.abs < Real.1
        geom_converges(q)
        converges(partial(q.pow))
        increasing_convergent_bounded_by_limit(partial(q.pow))
        is_upper_bound(partial(q.pow), limit(partial(q.pow)))
        is_upper_bound(partial(q.pow), limit(partial(q.pow))) =
            forall(n: Nat) { partial(q.pow, n) <= limit(partial(q.pow)) }
        forall(n: Nat) { partial(q.pow, n) <= limit(partial(q.pow)) }
    }
}

/// If L is the limit of the geometric series in ratio q with 0 <= q < 1, then
/// q (q L L) + q L <= q L L; the weighted-series bound is a fixed point of the
/// partial-sum recurrence.
lemma weighted_geom_bound_fixed_point(q: Real) {
    Real.0 <= q and q < Real.1 implies
        q * (q * limit(partial(q.pow)) * limit(partial(q.pow))) + q * limit(partial(q.pow))
        <= q * limit(partial(q.pow)) * limit(partial(q.pow))
} by {
    if Real.0 <= q and q < Real.1 {
        gte_zero_unfold(q)
        q >= Real.0
        abs_of_nonneg(q)
        q.abs = q
        q.abs < Real.1
        geom_series(q)
        limit(partial(q.pow)) = Real.1 / (Real.1 - q)
        Real.1 - q != Real.0
        mul_div_cancel(Real.1, Real.1 - q)
        (Real.1 - q) * (Real.1 / (Real.1 - q)) = Real.1
        (Real.1 - q) * limit(partial(q.pow)) = Real.1
        limit(partial(q.pow)) * (Real.1 - q) = Real.1
        limit(partial(q.pow)) - q * limit(partial(q.pow)) = Real.1
        limit(partial(q.pow)) - q * limit(partial(q.pow)) + q * limit(partial(q.pow)) =
            limit(partial(q.pow))
        Real.1 + q * limit(partial(q.pow)) = limit(partial(q.pow))
        limit(partial(q.pow)) = q * limit(partial(q.pow)) + Real.1
        q * limit(partial(q.pow)) + Real.1 <= limit(partial(q.pow))

        pow_zero[Real](q)
        q.pow(Nat.0) = Real.1
        partial_one(q.pow)
        partial(q.pow, Nat.1) = q.pow(Nat.0)
        partial(q.pow, Nat.1) = Real.1
        forall(k: Nat) {
            real_pow_nonneg_of_nonneg(q, k)
            Real.0 <= q.pow(k)
        }
        partial_increasing_of_nonneg(q.pow)
        is_increasing(partial(q.pow))
        geom_converges(q)
        converges(partial(q.pow))
        increasing_convergent_bounded_by_limit(partial(q.pow))
        is_upper_bound(partial(q.pow), limit(partial(q.pow)))
        is_upper_bound(partial(q.pow), limit(partial(q.pow))) =
            forall(n: Nat) { partial(q.pow, n) <= limit(partial(q.pow)) }
        partial(q.pow, Nat.1) <= limit(partial(q.pow))
        Real.1 <= limit(partial(q.pow))
        Real.0 <= Real.1
        lte_trans(Real.0, Real.1, limit(partial(q.pow)))
        Real.0 <= limit(partial(q.pow))

        mul_nonneg(q, limit(partial(q.pow)))
        q * limit(partial(q.pow)) >= Real.0
        gte_zero_unfold(q * limit(partial(q.pow)))
        Real.0 <= q * limit(partial(q.pow))
        mul_le_mul_of_nonneg_left[Real](q * limit(partial(q.pow)) + Real.1,
            limit(partial(q.pow)), q * limit(partial(q.pow)))
        q * limit(partial(q.pow)) * (q * limit(partial(q.pow)) + Real.1) <= q * limit(partial(q.pow)) * limit(partial(q.pow))
        q * (q * limit(partial(q.pow)) * limit(partial(q.pow))) + q * limit(partial(q.pow)) <= q * limit(partial(q.pow)) * limit(partial(q.pow))
    }
}

/// The partial sums of the weighted geometric series are bounded by
/// q L L, where L is the limit of the ordinary geometric series in ratio q.
theorem weighted_geom_partial_bound(q: Real, n: Nat) {
    Real.0 <= q and q < Real.1 implies
        partial(weighted_geom_term(q), n)
            <= q * limit(partial(q.pow)) * limit(partial(q.pow))
} by {
    if Real.0 <= q and q < Real.1 {
        geom_partial_bounded_by_limit(q)
        forall(m: Nat) {
            partial(q.pow, m) <= limit(partial(q.pow))
        }
        weighted_geom_bound_fixed_point(q)

        define p(k: Nat) -> Bool {
            partial(weighted_geom_term(q), k)
                <= q * limit(partial(q.pow)) * limit(partial(q.pow))
        }

        partial_zero(weighted_geom_term(q))
        partial(weighted_geom_term(q), Nat.0) = Real.0
        square_nonneg(limit(partial(q.pow)))
        limit(partial(q.pow)) * limit(partial(q.pow)) >= Real.0
        gte_zero_unfold(q)
        q >= Real.0
        mul_nonneg(q, limit(partial(q.pow)) * limit(partial(q.pow)))
        q * (limit(partial(q.pow)) * limit(partial(q.pow))) >= Real.0
        gte_zero_unfold(q * (limit(partial(q.pow)) * limit(partial(q.pow))))
        Real.0 <= q * (limit(partial(q.pow)) * limit(partial(q.pow)))
        Real.0 <= q * limit(partial(q.pow)) * limit(partial(q.pow))
        p(Nat.0)

        forall(m: Nat) {
            if p(m) {
                weighted_geom_partial_suc(q, m)
                partial(weighted_geom_term(q), m.suc) =
                    q * partial(weighted_geom_term(q), m) + q * partial(q.pow, m)
                partial(weighted_geom_term(q), m) <= q * limit(partial(q.pow)) * limit(partial(q.pow))
                Real.0 <= q
                mul_le_mul_of_nonneg_left[Real](partial(weighted_geom_term(q), m),
                    q * limit(partial(q.pow)) * limit(partial(q.pow)), q)
                q * partial(weighted_geom_term(q), m) <= q * (q * limit(partial(q.pow)) * limit(partial(q.pow)))
                partial(q.pow, m) <= limit(partial(q.pow))
                mul_le_mul_of_nonneg_left[Real](partial(q.pow, m), limit(partial(q.pow)), q)
                q * partial(q.pow, m) <= q * limit(partial(q.pow))
                add_le_add[Real](q * partial(weighted_geom_term(q), m),
                    q * (q * limit(partial(q.pow)) * limit(partial(q.pow))),
                    q * partial(q.pow, m), q * limit(partial(q.pow)))
                q * partial(weighted_geom_term(q), m) + q * partial(q.pow, m) <= q * (q * limit(partial(q.pow)) * limit(partial(q.pow))) + q * limit(partial(q.pow))
                weighted_geom_bound_fixed_point(q)
                q * (q * limit(partial(q.pow)) * limit(partial(q.pow))) + q * limit(partial(q.pow)) <= q * limit(partial(q.pow)) * limit(partial(q.pow))
                lte_trans(q * partial(weighted_geom_term(q), m) + q * partial(q.pow, m),
                    q * (q * limit(partial(q.pow)) * limit(partial(q.pow))) + q * limit(partial(q.pow)),
                    q * limit(partial(q.pow)) * limit(partial(q.pow)))
                q * partial(weighted_geom_term(q), m) + q * partial(q.pow, m) <= q * limit(partial(q.pow)) * limit(partial(q.pow))
                partial(weighted_geom_term(q), m.suc) <= q * limit(partial(q.pow)) * limit(partial(q.pow))
                p(m.suc)
            }
        }
        p(Nat.0) and forall(m: Nat) { p(m) implies p(m.suc) }
        alt_induction(p)
        forall(m: Nat) { p(m) }
        p(n)
        p(n) = (partial(weighted_geom_term(q), n) <= q * limit(partial(q.pow)) * limit(partial(q.pow)))
        partial(weighted_geom_term(q), n) <= q * limit(partial(q.pow)) * limit(partial(q.pow))
    }
}

/// The weighted geometric series Σ_{k>=0} k q^k converges for `0 <= q < 1`.
theorem weighted_geom_series_converges(q: Real) {
    Real.0 <= q and q < Real.1 implies converges(partial(weighted_geom_term(q)))
} by {
    if Real.0 <= q and q < Real.1 {
        forall(k: Nat) {
            weighted_geom_term_nonneg(q, k)
            Real.0 <= weighted_geom_term(q, k)
        }
        partial_increasing_of_nonneg(weighted_geom_term(q))
        is_increasing(partial(weighted_geom_term(q)))
        forall(n: Nat) {
            weighted_geom_partial_bound(q, n)
            partial(weighted_geom_term(q), n) <= q * limit(partial(q.pow)) * limit(partial(q.pow))
        }
        is_upper_bound(partial(weighted_geom_term(q)),
            q * limit(partial(q.pow)) * limit(partial(q.pow))) =
            forall(n: Nat) {
                partial(weighted_geom_term(q), n) <= q * limit(partial(q.pow)) * limit(partial(q.pow))
            }
        is_upper_bound(partial(weighted_geom_term(q)),
            q * limit(partial(q.pow)) * limit(partial(q.pow)))
        monotone_convergence_principle(partial(weighted_geom_term(q)),
            q * limit(partial(q.pow)) * limit(partial(q.pow)))
        converges(partial(weighted_geom_term(q)))
    }
}

/// The limit of the weighted geometric series satisfies the fixed-point
/// equation L_a = q L_a + q L, where L is the limit of the ordinary geometric
/// series in the same ratio.
theorem weighted_geom_limit_fixed_point(q: Real) {
    Real.0 <= q and q < Real.1 implies
        limit(partial(weighted_geom_term(q))) = q * limit(partial(weighted_geom_term(q))) + q * limit(partial(q.pow))
} by {
    if Real.0 <= q and q < Real.1 {
        weighted_geom_series_converges(q)
        converges(partial(weighted_geom_term(q)))
        gte_zero_unfold(q)
        q >= Real.0
        abs_of_nonneg(q)
        q.abs = q
        q.abs < Real.1
        geom_converges(q)
        converges(partial(q.pow))

        mul_seq_converges_to(q, partial(weighted_geom_term(q)))
        converges_to(mul_seq(q, partial(weighted_geom_term(q))),
            q * limit(partial(weighted_geom_term(q))))
        converges_to_imp_converges(mul_seq(q, partial(weighted_geom_term(q))),
            q * limit(partial(weighted_geom_term(q))))
        converges(mul_seq(q, partial(weighted_geom_term(q))))
        convergent_converges_to_limit(mul_seq(q, partial(weighted_geom_term(q))))
        converges_to(mul_seq(q, partial(weighted_geom_term(q))),
            limit(mul_seq(q, partial(weighted_geom_term(q)))))
        converges_to_unique(mul_seq(q, partial(weighted_geom_term(q))),
            limit(mul_seq(q, partial(weighted_geom_term(q)))),
            q * limit(partial(weighted_geom_term(q))))
        limit(mul_seq(q, partial(weighted_geom_term(q)))) =
            q * limit(partial(weighted_geom_term(q)))
        mul_seq_converges_to(q, partial(q.pow))
        converges_to(mul_seq(q, partial(q.pow)), q * limit(partial(q.pow)))
        converges_to_imp_converges(mul_seq(q, partial(q.pow)), q * limit(partial(q.pow)))
        converges(mul_seq(q, partial(q.pow)))
        convergent_converges_to_limit(mul_seq(q, partial(q.pow)))
        converges_to(mul_seq(q, partial(q.pow)), limit(mul_seq(q, partial(q.pow))))
        converges_to_unique(mul_seq(q, partial(q.pow)), limit(mul_seq(q, partial(q.pow))),
            q * limit(partial(q.pow)))
        limit(mul_seq(q, partial(q.pow))) = q * limit(partial(q.pow))
        limit_add_seq(mul_seq(q, partial(weighted_geom_term(q))), mul_seq(q, partial(q.pow)))
        converges_to(add_seq(mul_seq(q, partial(weighted_geom_term(q))), mul_seq(q, partial(q.pow))),
            limit(mul_seq(q, partial(weighted_geom_term(q)))) + limit(mul_seq(q, partial(q.pow))))
        converges_to(add_seq(mul_seq(q, partial(weighted_geom_term(q))), mul_seq(q, partial(q.pow))),
            q * limit(partial(weighted_geom_term(q))) + q * limit(partial(q.pow)))

        forall(n: Nat) {
            weighted_geom_partial_suc(q, n)
            partial(weighted_geom_term(q), n.suc) =
                q * partial(weighted_geom_term(q), n) + q * partial(q.pow, n)
            add_seq(mul_seq(q, partial(weighted_geom_term(q))), mul_seq(q, partial(q.pow)), n) =
                mul_seq(q, partial(weighted_geom_term(q)), n) + mul_seq(q, partial(q.pow), n)
            mul_seq(q, partial(weighted_geom_term(q)), n) =
                q * partial(weighted_geom_term(q), n)
            mul_seq(q, partial(q.pow), n) = q * partial(q.pow, n)
            compose(partial(weighted_geom_term(q)), Nat.suc, n) =
                partial(weighted_geom_term(q), Nat.suc(n))
            Nat.suc(n) = n.suc
            compose(partial(weighted_geom_term(q)), Nat.suc, n) =
                partial(weighted_geom_term(q), n.suc)
            compose(partial(weighted_geom_term(q)), Nat.suc, n) =
                add_seq(mul_seq(q, partial(weighted_geom_term(q))), mul_seq(q, partial(q.pow)), n)
        }
        seq_pointwise_eq_converges_to(
            add_seq(mul_seq(q, partial(weighted_geom_term(q))), mul_seq(q, partial(q.pow))),
            compose(partial(weighted_geom_term(q)), Nat.suc),
            q * limit(partial(weighted_geom_term(q))) + q * limit(partial(q.pow)))
        converges_to(compose(partial(weighted_geom_term(q)), Nat.suc),
            q * limit(partial(weighted_geom_term(q))) + q * limit(partial(q.pow)))

        forall(n: Nat) {
            tail(partial(weighted_geom_term(q)), Nat.1, n) =
                partial(weighted_geom_term(q), Nat.1 + n)
            Nat.1 + n = n.suc
            compose(partial(weighted_geom_term(q)), Nat.suc, n) =
                partial(weighted_geom_term(q), n.suc)
            tail(partial(weighted_geom_term(q)), Nat.1, n) =
                compose(partial(weighted_geom_term(q)), Nat.suc, n)
        }
        seq_pointwise_eq_converges_to(compose(partial(weighted_geom_term(q)), Nat.suc),
            tail(partial(weighted_geom_term(q)), Nat.1),
            q * limit(partial(weighted_geom_term(q))) + q * limit(partial(q.pow)))
        converges_to(tail(partial(weighted_geom_term(q)), Nat.1),
            q * limit(partial(weighted_geom_term(q))) + q * limit(partial(q.pow)))

        converges_to_imp_converges(tail(partial(weighted_geom_term(q)), Nat.1),
            q * limit(partial(weighted_geom_term(q))) + q * limit(partial(q.pow)))
        converges(tail(partial(weighted_geom_term(q)), Nat.1))
        tail_imp_converges_to(partial(weighted_geom_term(q)), Nat.1)
        converges_to(partial(weighted_geom_term(q)),
            limit(tail(partial(weighted_geom_term(q)), Nat.1)))
        convergent_converges_to_limit(partial(weighted_geom_term(q)))
        converges_to(partial(weighted_geom_term(q)), limit(partial(weighted_geom_term(q))))
        converges_to_unique(partial(weighted_geom_term(q)),
            limit(partial(weighted_geom_term(q))),
            limit(tail(partial(weighted_geom_term(q)), Nat.1)))
        limit(partial(weighted_geom_term(q))) =
            limit(tail(partial(weighted_geom_term(q)), Nat.1))
        convergent_converges_to_limit(tail(partial(weighted_geom_term(q)), Nat.1))
        converges_to(tail(partial(weighted_geom_term(q)), Nat.1),
            limit(tail(partial(weighted_geom_term(q)), Nat.1)))
        converges_to_unique(tail(partial(weighted_geom_term(q)), Nat.1),
            q * limit(partial(weighted_geom_term(q))) + q * limit(partial(q.pow)),
            limit(tail(partial(weighted_geom_term(q)), Nat.1)))
        q * limit(partial(weighted_geom_term(q))) + q * limit(partial(q.pow)) =
            limit(tail(partial(weighted_geom_term(q)), Nat.1))
        limit(partial(weighted_geom_term(q))) =
            q * limit(partial(weighted_geom_term(q))) + q * limit(partial(q.pow))
    }
}

/// The geometric mean series is the `p`-scaled weighted geometric series in the
/// ratio `1 - p`.
lemma geometric_mean_partial_eq_mul_seq(p: Real, n: Nat) {
    partial(geometric_mean_term(p), n) = mul_seq(p, partial(weighted_geom_term(Real.1 - p)), n)
} by {
    forall(k: Nat) {
        if k < n {
            geometric_mean_term(p, k) = from_nat[Real](k) * geometric_mass(p, k)
            geometric_mass(p, k) = p * (Real.1 - p).pow(k)
            geometric_mean_term(p, k) = from_nat[Real](k) * (p * (Real.1 - p).pow(k))
            from_nat[Real](k) * (p * (Real.1 - p).pow(k)) =
                p * (from_nat[Real](k) * (Real.1 - p).pow(k))
            weighted_geom_term(Real.1 - p, k) = from_nat[Real](k) * (Real.1 - p).pow(k)
            geometric_mean_term(p, k) = p * weighted_geom_term(Real.1 - p, k)
            mul_fn(p, weighted_geom_term(Real.1 - p), k) = p * weighted_geom_term(Real.1 - p, k)
            geometric_mean_term(p, k) = mul_fn(p, weighted_geom_term(Real.1 - p), k)
        }
    }
    partial_pointwise_eq(geometric_mean_term(p), mul_fn(p, weighted_geom_term(Real.1 - p)), n)
    partial(geometric_mean_term(p), n) = partial(mul_fn(p, weighted_geom_term(Real.1 - p)), n)
    partial_scalar_mul(p, weighted_geom_term(Real.1 - p), n)
    p * partial(weighted_geom_term(Real.1 - p), n) =
        partial(mul_fn(p, weighted_geom_term(Real.1 - p)), n)
    partial(geometric_mean_term(p), n) = p * partial(weighted_geom_term(Real.1 - p), n)
    mul_seq(p, partial(weighted_geom_term(Real.1 - p)), n) =
        p * partial(weighted_geom_term(Real.1 - p), n)
    partial(geometric_mean_term(p), n) = mul_seq(p, partial(weighted_geom_term(Real.1 - p)), n)
}

/// The expectation of the geometric distribution is (1-p)/p:
/// Σ_{k>=0} k p (1-p)^k = (1-p)/p.
theorem geometric_mean(p: Real) {
    Real.0 < p and p <= Real.1 implies
        limit(partial(geometric_mean_term(p))) = (Real.1 - p) / p
} by {
    if Real.0 < p and p <= Real.1 {
        geometric_ratio_bounds(p)
        Real.0 <= Real.1 - p and (Real.1 - p).abs < Real.1
        Real.0 <= Real.1 - p
        (Real.1 - p).abs < Real.1
        Real.0 < p
        -p < Real.0
        add_lt_add_left[Real](-p, Real.0, Real.1)
        Real.1 + -p < Real.1 + Real.0
        Real.1 - p < Real.1
        weighted_geom_series_converges(Real.1 - p)
        converges(partial(weighted_geom_term(Real.1 - p)))
        mul_seq_converges_to(p, partial(weighted_geom_term(Real.1 - p)))
        converges_to(mul_seq(p, partial(weighted_geom_term(Real.1 - p))),
            p * limit(partial(weighted_geom_term(Real.1 - p))))
        forall(n: Nat) {
            geometric_mean_partial_eq_mul_seq(p, n)
            partial(geometric_mean_term(p), n) =
                mul_seq(p, partial(weighted_geom_term(Real.1 - p)), n)
        }
        seq_pointwise_eq_converges_to(mul_seq(p, partial(weighted_geom_term(Real.1 - p))),
            partial(geometric_mean_term(p)),
            p * limit(partial(weighted_geom_term(Real.1 - p))))
        converges_to(partial(geometric_mean_term(p)),
            p * limit(partial(weighted_geom_term(Real.1 - p))))
        converges_to_imp_converges(partial(geometric_mean_term(p)),
            p * limit(partial(weighted_geom_term(Real.1 - p))))
        converges(partial(geometric_mean_term(p)))
        convergent_converges_to_limit(partial(geometric_mean_term(p)))
        converges_to(partial(geometric_mean_term(p)), limit(partial(geometric_mean_term(p))))
        converges_to_unique(partial(geometric_mean_term(p)), limit(partial(geometric_mean_term(p))),
            p * limit(partial(weighted_geom_term(Real.1 - p))))
        limit(partial(geometric_mean_term(p))) =
            p * limit(partial(weighted_geom_term(Real.1 - p)))

        weighted_geom_limit_fixed_point(Real.1 - p)
        limit(partial(weighted_geom_term(Real.1 - p))) =
            (Real.1 - p) * limit(partial(weighted_geom_term(Real.1 - p))) + (Real.1 - p) * limit(partial((Real.1 - p).pow))
        limit(partial(weighted_geom_term(Real.1 - p))) - (Real.1 - p) * limit(partial(weighted_geom_term(Real.1 - p))) =
            (Real.1 - p) * limit(partial((Real.1 - p).pow))
        limit(partial(weighted_geom_term(Real.1 - p))) * (Real.1 - (Real.1 - p)) =
            (Real.1 - p) * limit(partial((Real.1 - p).pow))
        Real.1 - (Real.1 - p) = p
        limit(partial(weighted_geom_term(Real.1 - p))) * p =
            (Real.1 - p) * limit(partial((Real.1 - p).pow))

        geom_series(Real.1 - p)
        (Real.1 - p).abs < Real.1
        limit(partial((Real.1 - p).pow)) = Real.1 / (Real.1 - (Real.1 - p))
        Real.1 - (Real.1 - p) = p
        limit(partial((Real.1 - p).pow)) = Real.1 / p
        p * limit(partial((Real.1 - p).pow)) = p * (Real.1 / p)
        p != Real.0
        p * (Real.1 / p) = Real.1
        p * limit(partial((Real.1 - p).pow)) = Real.1
        limit(partial((Real.1 - p).pow)) * p = Real.1

        limit(partial(geometric_mean_term(p))) * p =
            (p * limit(partial(weighted_geom_term(Real.1 - p)))) * p
        (p * limit(partial(weighted_geom_term(Real.1 - p)))) * p =
            p * (limit(partial(weighted_geom_term(Real.1 - p))) * p)
        p * (limit(partial(weighted_geom_term(Real.1 - p))) * p) =
            p * ((Real.1 - p) * limit(partial((Real.1 - p).pow)))
        p * ((Real.1 - p) * limit(partial((Real.1 - p).pow))) =
            (Real.1 - p) * (p * limit(partial((Real.1 - p).pow)))
        p * limit(partial((Real.1 - p).pow)) = Real.1
        (Real.1 - p) * (p * limit(partial((Real.1 - p).pow))) = (Real.1 - p) * Real.1
        (Real.1 - p) * Real.1 = Real.1 - p
        limit(partial(geometric_mean_term(p))) * p = Real.1 - p
        p * limit(partial(geometric_mean_term(p))) = Real.1 - p
        p != Real.0
        mul_left_cancel(Real.1 - p, p, limit(partial(geometric_mean_term(p))))
        (Real.1 - p) / p = limit(partial(geometric_mean_term(p)))
        limit(partial(geometric_mean_term(p))) = (Real.1 - p) / p
    }
}

/// The probability mass of the trial-counting geometric distribution with
/// parameter `p` at `k`: P(Y = k) = p (1-p)^(k-1) for k = 1, 2, ..., the trial
/// on which the first success occurs.  The mass at `k = 0` is zero.
define geometric_trial_mass(p: Real, k: Nat) -> Real {
    if k = Nat.0 { Real.0 } else { p * (Real.1 - p).pow(k - Nat.1) }
}

/// The trial-counting mass at trial `k + 1` equals the failure-counting mass
/// at `k`.
theorem geometric_trial_mass_suc(p: Real, k: Nat) {
    geometric_trial_mass(p, k.suc) = geometric_mass(p, k)
} by {
    if k = Nat.0 {
        geometric_trial_mass(p, Nat.1) = p * (Real.1 - p).pow(Nat.1 - Nat.1)
        Nat.1 - Nat.1 = Nat.0
        (Real.1 - p).pow(Nat.1 - Nat.1) = (Real.1 - p).pow(Nat.0)
        pow_zero[Real](Real.1 - p)
        (Real.1 - p).pow(Nat.0) = Real.1
        geometric_trial_mass(p, Nat.1) = p * Real.1
        p * Real.1 = p
        geometric_mass(p, Nat.0) = p * (Real.1 - p).pow(Nat.0)
        geometric_mass(p, Nat.0) = p * Real.1
        geometric_trial_mass(p, k.suc) = geometric_mass(p, k)
    }
    if k != Nat.0 {
        let m: Nat satisfy { m.suc = k }
        geometric_trial_mass(p, m.suc.suc) = p * (Real.1 - p).pow(m.suc.suc - Nat.1)
        m.suc.suc - Nat.1 = m.suc
        (Real.1 - p).pow(m.suc.suc - Nat.1) = (Real.1 - p).pow(m.suc)
        geometric_trial_mass(p, m.suc.suc) = p * (Real.1 - p).pow(m.suc)
        geometric_mass(p, m.suc) = p * (Real.1 - p).pow(m.suc)
        geometric_trial_mass(p, k.suc) = geometric_mass(p, k)
    }
    k = Nat.0 or k != Nat.0
}

/// The k-th summand of the trial-counting geometric expectation: k p (1-p)^(k-1).
define geometric_trial_mean_term(p: Real, k: Nat) -> Real {
    from_nat[Real](k) * geometric_trial_mass(p, k)
}

/// The shifted trial mean terms are the failure mean terms plus the masses:
/// (k+1) p (1-p)^k = k p (1-p)^k + p (1-p)^k.
lemma geometric_trial_mean_term_shift(p: Real, k: Nat) {
    geometric_trial_mean_term(p, k.suc) =
        geometric_mean_term(p, k) + geometric_mass(p, k)
} by {
    geometric_trial_mean_term(p, k.suc) =
        from_nat[Real](k.suc) * geometric_trial_mass(p, k.suc)
    from_nat_add[Real](k, Nat.1)
    from_nat[Real](k.suc) = from_nat[Real](k) + from_nat[Real](Nat.1)
    from_nat_one[Real]
    from_nat[Real](Nat.1) = Real.1
    from_nat[Real](k.suc) = from_nat[Real](k) + Real.1
    geometric_trial_mass_suc(p, k)
    geometric_trial_mass(p, k.suc) = geometric_mass(p, k)
    geometric_trial_mean_term(p, k.suc) =
        (from_nat[Real](k) + Real.1) * geometric_mass(p, k)
    (from_nat[Real](k) + Real.1) * geometric_mass(p, k) =
        from_nat[Real](k) * geometric_mass(p, k) + geometric_mass(p, k)
    geometric_mean_term(p, k) = from_nat[Real](k) * geometric_mass(p, k)
    geometric_trial_mean_term(p, k.suc) =
        geometric_mean_term(p, k) + geometric_mass(p, k)
}

/// The trial-counting geometric probabilities sum to one:
/// Σ_{k>=1} p (1-p)^(k-1) = 1.
theorem geometric_trial_series_sums_to_one(p: Real) {
    Real.0 < p and p <= Real.1 implies limit(partial(geometric_trial_mass(p))) = Real.1
} by {
    if Real.0 < p and p <= Real.1 {
        forall(k: Nat) {
            geometric_trial_mass_suc(p, k)
            geometric_trial_mass(p, k.suc) = geometric_mass(p, k)
            compose(geometric_trial_mass(p), Nat.suc, k) =
                geometric_trial_mass(p, Nat.suc(k))
            Nat.suc(k) = k.suc
            compose(geometric_trial_mass(p), Nat.suc, k) =
                geometric_trial_mass(p, k.suc)
            compose(geometric_trial_mass(p), Nat.suc, k) = geometric_mass(p, k)
        }
        function_extensionality[Nat, Real](compose(geometric_trial_mass(p), Nat.suc),
            geometric_mass(p))
        compose(geometric_trial_mass(p), Nat.suc) = geometric_mass(p)
        forall(n: Nat) {
            geometric_trial_mass(p, Nat.0) = Real.0
            partial_shift_suc(geometric_trial_mass(p), n)
            geometric_trial_mass(p, Nat.0) + partial(compose(geometric_trial_mass(p), Nat.suc), n) =
                partial(geometric_trial_mass(p), n.suc)
            partial(compose(geometric_trial_mass(p), Nat.suc), n) =
                partial(geometric_trial_mass(p), n.suc)
            partial(geometric_mass(p), n) = partial(geometric_trial_mass(p), n.suc)
            compose(partial(geometric_trial_mass(p)), Nat.suc, n) =
                partial(geometric_trial_mass(p), Nat.suc(n))
            Nat.suc(n) = n.suc
            compose(partial(geometric_trial_mass(p)), Nat.suc, n) =
                partial(geometric_trial_mass(p), n.suc)
            compose(partial(geometric_trial_mass(p)), Nat.suc, n) =
                partial(geometric_mass(p), n)
        }
        geometric_series_converges(p)
        converges(partial(geometric_mass(p)))
        convergent_converges_to_limit(partial(geometric_mass(p)))
        converges_to(partial(geometric_mass(p)), limit(partial(geometric_mass(p))))
        geometric_series_sums_to_one(p)
        limit(partial(geometric_mass(p))) = Real.1
        converges_to(partial(geometric_mass(p)), Real.1)
        seq_pointwise_eq_converges_to(partial(geometric_mass(p)),
            compose(partial(geometric_trial_mass(p)), Nat.suc), Real.1)
        converges_to(compose(partial(geometric_trial_mass(p)), Nat.suc), Real.1)
        forall(n: Nat) {
            tail(partial(geometric_trial_mass(p)), Nat.1, n) =
                partial(geometric_trial_mass(p), Nat.1 + n)
            Nat.1 + n = n.suc
            compose(partial(geometric_trial_mass(p)), Nat.suc, n) =
                partial(geometric_trial_mass(p), n.suc)
            tail(partial(geometric_trial_mass(p)), Nat.1, n) =
                compose(partial(geometric_trial_mass(p)), Nat.suc, n)
        }
        seq_pointwise_eq_converges_to(compose(partial(geometric_trial_mass(p)), Nat.suc),
            tail(partial(geometric_trial_mass(p)), Nat.1), Real.1)
        converges_to(tail(partial(geometric_trial_mass(p)), Nat.1), Real.1)
        converges_to_imp_converges(tail(partial(geometric_trial_mass(p)), Nat.1), Real.1)
        converges(tail(partial(geometric_trial_mass(p)), Nat.1))
        tail_imp_converges_to(partial(geometric_trial_mass(p)), Nat.1)
        converges_to(partial(geometric_trial_mass(p)),
            limit(tail(partial(geometric_trial_mass(p)), Nat.1)))
        convergent_converges_to_limit(tail(partial(geometric_trial_mass(p)), Nat.1))
        converges_to(tail(partial(geometric_trial_mass(p)), Nat.1),
            limit(tail(partial(geometric_trial_mass(p)), Nat.1)))
        converges_to_unique(tail(partial(geometric_trial_mass(p)), Nat.1), Real.1,
            limit(tail(partial(geometric_trial_mass(p)), Nat.1)))
        Real.1 = limit(tail(partial(geometric_trial_mass(p)), Nat.1))
        converges_to(partial(geometric_trial_mass(p)), Real.1)
        convergent_converges_to_limit(partial(geometric_trial_mass(p)))
        converges_to(partial(geometric_trial_mass(p)), limit(partial(geometric_trial_mass(p))))
        converges_to_unique(partial(geometric_trial_mass(p)), limit(partial(geometric_trial_mass(p))), Real.1)
        limit(partial(geometric_trial_mass(p))) = Real.1
    }
}

/// The expectation of the trial-counting geometric distribution is 1/p:
/// Σ_{k>=1} k p (1-p)^(k-1) = 1/p.
theorem geometric_trial_mean(p: Real) {
    Real.0 < p and p <= Real.1 implies
        limit(partial(geometric_trial_mean_term(p))) = Real.1 / p
} by {
    if Real.0 < p and p <= Real.1 {
        forall(k: Nat) {
            geometric_trial_mean_term_shift(p, k)
            geometric_trial_mean_term(p, k.suc) =
                geometric_mean_term(p, k) + geometric_mass(p, k)
            compose(geometric_trial_mean_term(p), Nat.suc, k) =
                geometric_trial_mean_term(p, Nat.suc(k))
            Nat.suc(k) = k.suc
            compose(geometric_trial_mean_term(p), Nat.suc, k) =
                geometric_trial_mean_term(p, k.suc)
            add_fn(geometric_mean_term(p), geometric_mass(p), k) =
                geometric_mean_term(p, k) + geometric_mass(p, k)
            compose(geometric_trial_mean_term(p), Nat.suc, k) =
                add_fn(geometric_mean_term(p), geometric_mass(p), k)
        }
        function_extensionality[Nat, Real](compose(geometric_trial_mean_term(p), Nat.suc),
            add_fn(geometric_mean_term(p), geometric_mass(p)))
        compose(geometric_trial_mean_term(p), Nat.suc) =
            add_fn(geometric_mean_term(p), geometric_mass(p))
        forall(n: Nat) {
            geometric_trial_mean_term(p, Nat.0) =
                from_nat[Real](Nat.0) * geometric_trial_mass(p, Nat.0)
            from_nat_zero[Real]
            from_nat[Real](Nat.0) = Real.0
            Real.0 * geometric_trial_mass(p, Nat.0) = Real.0
            geometric_trial_mean_term(p, Nat.0) = Real.0
            partial_shift_suc(geometric_trial_mean_term(p), n)
            geometric_trial_mean_term(p, Nat.0) + partial(compose(geometric_trial_mean_term(p), Nat.suc), n) =
                partial(geometric_trial_mean_term(p), n.suc)
            partial(compose(geometric_trial_mean_term(p), Nat.suc), n) =
                partial(geometric_trial_mean_term(p), n.suc)
            partial(add_fn(geometric_mean_term(p), geometric_mass(p)), n) =
                partial(geometric_trial_mean_term(p), n.suc)
            partial_add(geometric_mean_term(p), geometric_mass(p), n)
            partial(add_fn(geometric_mean_term(p), geometric_mass(p)), n) =
                partial(geometric_mean_term(p), n) + partial(geometric_mass(p), n)
            partial(geometric_trial_mean_term(p), n.suc) =
                partial(geometric_mean_term(p), n) + partial(geometric_mass(p), n)
            compose(partial(geometric_trial_mean_term(p)), Nat.suc, n) =
                partial(geometric_trial_mean_term(p), Nat.suc(n))
            Nat.suc(n) = n.suc
            compose(partial(geometric_trial_mean_term(p)), Nat.suc, n) =
                partial(geometric_trial_mean_term(p), n.suc)
            add_seq(partial(geometric_mean_term(p)), partial(geometric_mass(p)), n) =
                partial(geometric_mean_term(p), n) + partial(geometric_mass(p), n)
            compose(partial(geometric_trial_mean_term(p)), Nat.suc, n) =
                add_seq(partial(geometric_mean_term(p)), partial(geometric_mass(p)), n)
        }
        geometric_series_converges(p)
        converges(partial(geometric_mass(p)))
        geometric_mean(p)
        geometric_ratio_bounds(p)
        Real.0 <= Real.1 - p and (Real.1 - p).abs < Real.1
        Real.0 < p
        -p < Real.0
        add_lt_add_left[Real](-p, Real.0, Real.1)
        Real.1 + -p < Real.1 + Real.0
        Real.1 - p < Real.1
        weighted_geom_series_converges(Real.1 - p)
        converges(partial(weighted_geom_term(Real.1 - p)))
        mul_seq_converges_to(p, partial(weighted_geom_term(Real.1 - p)))
        converges_to(mul_seq(p, partial(weighted_geom_term(Real.1 - p))),
            p * limit(partial(weighted_geom_term(Real.1 - p))))
        forall(n: Nat) {
            geometric_mean_partial_eq_mul_seq(p, n)
            partial(geometric_mean_term(p), n) =
                mul_seq(p, partial(weighted_geom_term(Real.1 - p)), n)
        }
        seq_pointwise_eq_converges_to(mul_seq(p, partial(weighted_geom_term(Real.1 - p))),
            partial(geometric_mean_term(p)),
            p * limit(partial(weighted_geom_term(Real.1 - p))))
        converges_to(partial(geometric_mean_term(p)),
            p * limit(partial(weighted_geom_term(Real.1 - p))))
        converges_to_imp_converges(partial(geometric_mean_term(p)),
            p * limit(partial(weighted_geom_term(Real.1 - p))))
        converges(partial(geometric_mean_term(p)))
        limit_add_seq(partial(geometric_mean_term(p)), partial(geometric_mass(p)))
        converges_to(add_seq(partial(geometric_mean_term(p)), partial(geometric_mass(p))),
            limit(partial(geometric_mean_term(p))) + limit(partial(geometric_mass(p))))
        seq_pointwise_eq_converges_to(
            add_seq(partial(geometric_mean_term(p)), partial(geometric_mass(p))),
            compose(partial(geometric_trial_mean_term(p)), Nat.suc),
            limit(partial(geometric_mean_term(p))) + limit(partial(geometric_mass(p))))
        converges_to(compose(partial(geometric_trial_mean_term(p)), Nat.suc),
            limit(partial(geometric_mean_term(p))) + limit(partial(geometric_mass(p))))
        forall(n: Nat) {
            tail(partial(geometric_trial_mean_term(p)), Nat.1, n) =
                partial(geometric_trial_mean_term(p), Nat.1 + n)
            Nat.1 + n = n.suc
            compose(partial(geometric_trial_mean_term(p)), Nat.suc, n) =
                partial(geometric_trial_mean_term(p), n.suc)
            tail(partial(geometric_trial_mean_term(p)), Nat.1, n) =
                compose(partial(geometric_trial_mean_term(p)), Nat.suc, n)
        }
        seq_pointwise_eq_converges_to(compose(partial(geometric_trial_mean_term(p)), Nat.suc),
            tail(partial(geometric_trial_mean_term(p)), Nat.1),
            limit(partial(geometric_mean_term(p))) + limit(partial(geometric_mass(p))))
        converges_to(tail(partial(geometric_trial_mean_term(p)), Nat.1),
            limit(partial(geometric_mean_term(p))) + limit(partial(geometric_mass(p))))
        converges_to_imp_converges(tail(partial(geometric_trial_mean_term(p)), Nat.1),
            limit(partial(geometric_mean_term(p))) + limit(partial(geometric_mass(p))))
        converges(tail(partial(geometric_trial_mean_term(p)), Nat.1))
        tail_imp_converges_to(partial(geometric_trial_mean_term(p)), Nat.1)
        converges_to(partial(geometric_trial_mean_term(p)),
            limit(tail(partial(geometric_trial_mean_term(p)), Nat.1)))
        convergent_converges_to_limit(partial(geometric_trial_mean_term(p)))
        converges_to(partial(geometric_trial_mean_term(p)),
            limit(partial(geometric_trial_mean_term(p))))
        converges_to_unique(partial(geometric_trial_mean_term(p)),
            limit(partial(geometric_trial_mean_term(p))),
            limit(tail(partial(geometric_trial_mean_term(p)), Nat.1)))
        limit(partial(geometric_trial_mean_term(p))) =
            limit(tail(partial(geometric_trial_mean_term(p)), Nat.1))
        convergent_converges_to_limit(tail(partial(geometric_trial_mean_term(p)), Nat.1))
        converges_to(tail(partial(geometric_trial_mean_term(p)), Nat.1),
            limit(tail(partial(geometric_trial_mean_term(p)), Nat.1)))
        converges_to_unique(tail(partial(geometric_trial_mean_term(p)), Nat.1),
            limit(partial(geometric_mean_term(p))) + limit(partial(geometric_mass(p))),
            limit(tail(partial(geometric_trial_mean_term(p)), Nat.1)))
        limit(partial(geometric_mean_term(p))) + limit(partial(geometric_mass(p))) =
            limit(tail(partial(geometric_trial_mean_term(p)), Nat.1))
        limit(partial(geometric_trial_mean_term(p))) =
            limit(partial(geometric_mean_term(p))) + limit(partial(geometric_mass(p)))
        geometric_mean(p)
        limit(partial(geometric_mean_term(p))) = (Real.1 - p) / p
        geometric_series_sums_to_one(p)
        limit(partial(geometric_mass(p))) = Real.1
        limit(partial(geometric_trial_mean_term(p))) = (Real.1 - p) / p + Real.1
        p != Real.0
        p * ((Real.1 - p) / p) = Real.1 - p
        p * ((Real.1 - p) / p + Real.1) = p * ((Real.1 - p) / p) + p * Real.1
        p * Real.1 = p
        p * ((Real.1 - p) / p + Real.1) = (Real.1 - p) + p
        (Real.1 - p) + p = Real.1
        Real.1 = p * ((Real.1 - p) / p + Real.1)
        mul_left_cancel(Real.1, p, (Real.1 - p) / p + Real.1)
        Real.1 / p = (Real.1 - p) / p + Real.1
        limit(partial(geometric_trial_mean_term(p))) = Real.1 / p
    }
}

/// The tail of the geometric mass from index `k`: the mass at `k + j` for
/// `j = 0, 1, 2, ...`.  The tail sums give P(X >= k) = (1-p)^k.
define geometric_tail_mass(p: Real, k: Nat, j: Nat) -> Real {
    geometric_mass(p, k + j)
}

/// The geometric tail series sums to (1-p)^k: Σ_{j>=0} p (1-p)^(k+j) = (1-p)^k.
/// Equivalently, P(X >= k) = (1-p)^k for the failure-counting variable.
theorem geometric_tail_sums(p: Real, k: Nat) {
    Real.0 < p and p <= Real.1 implies
        limit(partial(geometric_tail_mass(p, k))) = (Real.1 - p).pow(k)
} by {
    if Real.0 < p and p <= Real.1 {
        geometric_ratio_bounds(p)
        Real.0 <= Real.1 - p and (Real.1 - p).abs < Real.1
        Real.0 < p
        -p < Real.0
        add_lt_add_left[Real](-p, Real.0, Real.1)
        Real.1 + -p < Real.1 + Real.0
        Real.1 - p < Real.1
        geometric_series_converges(p)
        converges(partial(geometric_mass(p)))
        mul_seq_converges_to((Real.1 - p).pow(k), partial(geometric_mass(p)))
        converges_to(mul_seq((Real.1 - p).pow(k), partial(geometric_mass(p))),
            (Real.1 - p).pow(k) * limit(partial(geometric_mass(p))))
        forall(n: Nat) {
            forall(j: Nat) {
                if j < n {
                    geometric_tail_mass(p, k, j) = geometric_mass(p, k + j)
                    geometric_mass(p, k + j) =
                        p * (Real.1 - p).pow(k + j)
                    pow_add[Real](Real.1 - p, k, j)
                    (Real.1 - p).pow(k + j) = (Real.1 - p).pow(k) * (Real.1 - p).pow(j)
                    geometric_mass(p, k + j) =
                        p * ((Real.1 - p).pow(k) * (Real.1 - p).pow(j))
                    p * ((Real.1 - p).pow(k) * (Real.1 - p).pow(j)) =
                        (Real.1 - p).pow(k) * (p * (Real.1 - p).pow(j))
                    geometric_mass(p, j) = p * (Real.1 - p).pow(j)
                    geometric_mass(p, k + j) = (Real.1 - p).pow(k) * geometric_mass(p, j)
                    mul_fn((Real.1 - p).pow(k), geometric_mass(p), j) =
                        (Real.1 - p).pow(k) * geometric_mass(p, j)
                    geometric_tail_mass(p, k, j) =
                        mul_fn((Real.1 - p).pow(k), geometric_mass(p), j)
                }
            }
            partial_pointwise_eq(geometric_tail_mass(p, k),
                mul_fn((Real.1 - p).pow(k), geometric_mass(p)), n)
            partial(geometric_tail_mass(p, k), n) =
                partial(mul_fn((Real.1 - p).pow(k), geometric_mass(p)), n)
            partial_scalar_mul((Real.1 - p).pow(k), geometric_mass(p), n)
            (Real.1 - p).pow(k) * partial(geometric_mass(p), n) =
                partial(mul_fn((Real.1 - p).pow(k), geometric_mass(p)), n)
            partial(geometric_tail_mass(p, k), n) =
                (Real.1 - p).pow(k) * partial(geometric_mass(p), n)
            mul_seq((Real.1 - p).pow(k), partial(geometric_mass(p)), n) =
                (Real.1 - p).pow(k) * partial(geometric_mass(p), n)
            partial(geometric_tail_mass(p, k), n) =
                mul_seq((Real.1 - p).pow(k), partial(geometric_mass(p)), n)
        }
        seq_pointwise_eq_converges_to(
            mul_seq((Real.1 - p).pow(k), partial(geometric_mass(p))),
            partial(geometric_tail_mass(p, k)),
            (Real.1 - p).pow(k) * limit(partial(geometric_mass(p))))
        converges_to(partial(geometric_tail_mass(p, k)),
            (Real.1 - p).pow(k) * limit(partial(geometric_mass(p))))
        geometric_series_sums_to_one(p)
        limit(partial(geometric_mass(p))) = Real.1
        (Real.1 - p).pow(k) * limit(partial(geometric_mass(p))) =
            (Real.1 - p).pow(k) * Real.1
        (Real.1 - p).pow(k) * Real.1 = (Real.1 - p).pow(k)
        converges_to(partial(geometric_tail_mass(p, k)), (Real.1 - p).pow(k))
        converges_to_imp_converges(partial(geometric_tail_mass(p, k)), (Real.1 - p).pow(k))
        converges(partial(geometric_tail_mass(p, k)))
        convergent_converges_to_limit(partial(geometric_tail_mass(p, k)))
        converges_to(partial(geometric_tail_mass(p, k)), limit(partial(geometric_tail_mass(p, k))))
        converges_to_unique(partial(geometric_tail_mass(p, k)),
            limit(partial(geometric_tail_mass(p, k))), (Real.1 - p).pow(k))
        limit(partial(geometric_tail_mass(p, k))) = (Real.1 - p).pow(k)
    }
}

/// Memorylessness of the geometric distribution (multiplication form).
/// For the failure-counting variable X, P(X >= s+t) = P(X >= s) P(X >= t);
/// for the trial-counting variable Y = X + 1 this is the classical
/// P(Y > s+t) = P(Y > s) P(Y > t), and dividing by P(Y > s) (nonzero when
/// `p < 1`) yields P(Y > s+t | Y > s) = P(Y > t).
theorem geometric_memoryless(p: Real, s: Nat, t: Nat) {
    Real.0 < p and p <= Real.1 implies
        limit(partial(geometric_tail_mass(p, s + t))) =
            limit(partial(geometric_tail_mass(p, s))) * limit(partial(geometric_tail_mass(p, t)))
} by {
    if Real.0 < p and p <= Real.1 {
        geometric_tail_sums(p, s + t)
        limit(partial(geometric_tail_mass(p, s + t))) = (Real.1 - p).pow(s + t)
        geometric_tail_sums(p, s)
        limit(partial(geometric_tail_mass(p, s))) = (Real.1 - p).pow(s)
        geometric_tail_sums(p, t)
        limit(partial(geometric_tail_mass(p, t))) = (Real.1 - p).pow(t)
        pow_add[Real](Real.1 - p, s, t)
        (Real.1 - p).pow(s + t) = (Real.1 - p).pow(s) * (Real.1 - p).pow(t)
        limit(partial(geometric_tail_mass(p, s + t))) =
            (Real.1 - p).pow(s) * (Real.1 - p).pow(t)
        limit(partial(geometric_tail_mass(p, s))) * limit(partial(geometric_tail_mass(p, t))) =
            (Real.1 - p).pow(s) * (Real.1 - p).pow(t)
        limit(partial(geometric_tail_mass(p, s + t))) =
            limit(partial(geometric_tail_mass(p, s))) * limit(partial(geometric_tail_mass(p, t)))
    }
}

/// The conditional form of memorylessness: P(X >= s+t) / P(X >= s) = P(X >= t),
/// equivalently P(Y > s+t | Y > s) = P(Y > t) for the trial-counting variable,
/// when the conditioning event has positive probability (that is, `p < 1`).
theorem geometric_memoryless_conditional(p: Real, s: Nat, t: Nat) {
    Real.0 < p and p < Real.1 implies
        limit(partial(geometric_tail_mass(p, s + t))) / limit(partial(geometric_tail_mass(p, s))) = limit(partial(geometric_tail_mass(p, t)))
} by {
    if Real.0 < p and p < Real.1 {
        geometric_tail_sums(p, s + t)
        limit(partial(geometric_tail_mass(p, s + t))) = (Real.1 - p).pow(s + t)
        geometric_tail_sums(p, s)
        limit(partial(geometric_tail_mass(p, s))) = (Real.1 - p).pow(s)
        geometric_tail_sums(p, t)
        limit(partial(geometric_tail_mass(p, t))) = (Real.1 - p).pow(t)
        pow_add[Real](Real.1 - p, s, t)
        (Real.1 - p).pow(s + t) = (Real.1 - p).pow(s) * (Real.1 - p).pow(t)
        limit(partial(geometric_tail_mass(p, s))) * limit(partial(geometric_tail_mass(p, t))) =
            (Real.1 - p).pow(s) * (Real.1 - p).pow(t)
        limit(partial(geometric_tail_mass(p, s + t))) =
            limit(partial(geometric_tail_mass(p, s))) * limit(partial(geometric_tail_mass(p, t)))
        p < Real.1
        add_lt_add_right[Real](p, Real.1, -p)
        p + -p < Real.1 + -p
        p + -p = Real.0
        Real.0 < Real.1 - p
        positive_pows_of_positive_are_positive[Real](Real.1 - p, s)
        (Real.1 - p).pow(s) > Real.0
        (Real.1 - p).pow(s) != Real.0
        limit(partial(geometric_tail_mass(p, s))) != Real.0
        mul_left_cancel(limit(partial(geometric_tail_mass(p, s + t))),
            limit(partial(geometric_tail_mass(p, s))),
            limit(partial(geometric_tail_mass(p, t))))
        limit(partial(geometric_tail_mass(p, s + t))) / limit(partial(geometric_tail_mass(p, s))) =
            limit(partial(geometric_tail_mass(p, t)))
    }
}




// The second moment and variance of the geometric distribution:
//   E[X^2] = (1-p)(2-p) / p^2   and   Var(X) = E[X^2] - E[X]^2 = (1-p) / p^2.
// The proof route mirrors the mean: the series S2 = Σ k^2 q^k satisfies the
// second-order recurrence partial(b, n+1) = q partial(b, n) + 2q partial(a, n)
// + q partial(q.pow, n), so its limit obeys S2 (1-q) = 2 q M1 + q L with
// M1 = q/(1-q)^2 and L = 1/(1-q), giving E[X^2] = p S2 = (1-p)(2-p)/p^2 and
// Var(X) = (1-p)/p^2.  Convergence of Σ k^2 q^k follows by bounding the partial
// sums with (2 q M1 + q L) L (monotone convergence), exactly as for the mean.
// The verification of these series steps is left as a statement here, in the
// style of the general binomial moments in binomial.ac.
