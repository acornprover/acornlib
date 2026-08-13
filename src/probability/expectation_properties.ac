from real import Real, div_le_of_mul_le
from algebra.add_semigroup import add_fn
from algebra.semigroup import mul_fn
from finite_set import FiniteSet
from data.finite.finite_set_filter import finite_set_filter, finite_set_filter_contains_eq
from data.basic.functions import identity_fn
from probability.discrete_pmf import DiscretePMF, discrete_expectation, discrete_expectation_add,
    discrete_expectation_scalar_mul, discrete_expectation_const, discrete_expectation_monotone,
    discrete_expectation_indicator_eq_event_prob, discrete_expectation_linear_combo,
    finite_event_indicator, finite_event_indicator_one_of_contains,
    finite_event_indicator_zero_of_not_contains, const_real_fn, pmf_event_prob
from probability.discrete_probability_ext import discrete_rv_independent
from probability.discrete_dirac_pmf import dirac_pmf, dirac_expectation

/// The expectation of a constant random variable is the constant, under any discrete pmf.
/// (This restates `discrete_expectation_const` from `discrete_pmf` for the constant
/// random variable `X(t) = c`.)
theorem expectation_const[T](pmf: DiscretePMF[T], c: Real) {
    discrete_expectation(pmf, const_real_fn[T](c)) = c
} by {
    discrete_expectation_const[T](pmf, c)
}

/// Under the Dirac pmf at `x`, the expectation of a random variable is its value at `x`,
/// so the expectation of the identity random variable is `x` itself.
theorem dirac_expectation_identity(x: Real) {
    discrete_expectation(dirac_pmf[Real](x), identity_fn[Real]) = x
} by {
    dirac_expectation[Real](x, identity_fn[Real])
    identity_fn[Real](x) = x
    discrete_expectation(dirac_pmf[Real](x), identity_fn[Real]) = x
}

/// Under the Dirac pmf at `c`, the constant random variable taking the value `c` has
/// expectation `c`.
theorem dirac_expectation_const(c: Real) {
    discrete_expectation(dirac_pmf[Real](c), const_real_fn[Real](c)) = c
} by {
    dirac_expectation[Real](c, const_real_fn[Real](c))
    discrete_expectation(dirac_pmf[Real](c), const_real_fn[Real](c)) = const_real_fn[Real](c, c)
    const_real_fn[Real](c, c) = c
    discrete_expectation(dirac_pmf[Real](c), const_real_fn[Real](c)) = c
}

/// Expectation is affine in the random variable:
/// `E(a * X + b) = a * E(X) + b` for real constants `a` and `b`,
/// combining the scalar, additivity, and constant laws.
theorem discrete_expectation_affine[T](pmf: DiscretePMF[T], a: Real, x: T -> Real, b: Real) {
    discrete_expectation(pmf, add_fn(mul_fn(a, x), const_real_fn[T](b)))
        = a * discrete_expectation(pmf, x) + b
} by {
    discrete_expectation_add[T](pmf, mul_fn(a, x), const_real_fn[T](b))
    discrete_expectation(pmf, add_fn(mul_fn(a, x), const_real_fn[T](b))) =
        discrete_expectation(pmf, mul_fn(a, x)) + discrete_expectation(pmf, const_real_fn[T](b))
    discrete_expectation_scalar_mul[T](pmf, a, x)
    discrete_expectation(pmf, mul_fn(a, x)) = a * discrete_expectation(pmf, x)
    discrete_expectation_const[T](pmf, b)
    discrete_expectation(pmf, const_real_fn[T](b)) = b
}

/// Expectation is linear in a two-term combination:
/// `E(a * X + b * Y) = a * E(X) + b * E(Y)` for real constants `a` and `b`.
/// This holds for arbitrary (not necessarily independent) random variables,
/// since expectation is additive and homogeneous in the discrete framework.
theorem expectation_linear_combo[T](pmf: DiscretePMF[T], a: Real, x: T -> Real, b: Real, y: T -> Real) {
    discrete_expectation(pmf, add_fn(mul_fn(a, x), mul_fn(b, y)))
        = a * discrete_expectation(pmf, x) + b * discrete_expectation(pmf, y)
} by {
    discrete_expectation_linear_combo[T](pmf, a, x, b, y)
}

/// Expectation is additive over the sum of two independent random variables:
/// `E(X + Y) = E(X) + E(Y)`.  Linearity of expectation holds without independence;
/// the hypothesis is kept to match the classical statement.
theorem discrete_expectation_add_independent[T](pmf: DiscretePMF[T], x: T -> Real, y: T -> Real) {
    discrete_rv_independent(pmf, x, y) implies
        discrete_expectation(pmf, add_fn(x, y)) = discrete_expectation(pmf, x) + discrete_expectation(pmf, y)
} by {
    if discrete_rv_independent(pmf, x, y) {
        discrete_expectation_add[T](pmf, x, y)
    }
}

/// The predicate that a random variable takes a value of at least `a`.
define rv_ge_contains[T](rv: T -> Real, a: Real, x: T) -> Bool {
    a <= rv(x)
}

/// The event that the random variable is at least `a`, restricted to the pmf support.
define rv_ge_event[T](pmf: DiscretePMF[T], rv: T -> Real, a: Real) -> FiniteSet[T] {
    finite_set_filter(pmf.support, rv_ge_contains(rv, a))
}

/// Membership in the `X >= a` event is support membership together with `a <= X`.
theorem rv_ge_event_contains_eq[T](pmf: DiscretePMF[T], rv: T -> Real, a: Real, x: T) {
    rv_ge_event(pmf, rv, a).contains(x) = (pmf.support.contains(x) and a <= rv(x))
} by {
    rv_ge_contains(rv, a, x) = (a <= rv(x))
    finite_set_filter_contains_eq(pmf.support, rv_ge_contains(rv, a), x)
    finite_set_filter(pmf.support, rv_ge_contains(rv, a)).contains(x) =
        (pmf.support.contains(x) and rv_ge_contains(rv, a, x))
    finite_set_filter(pmf.support, rv_ge_contains(rv, a)).contains(x) =
        (pmf.support.contains(x) and a <= rv(x))
    rv_ge_event(pmf, rv, a) = finite_set_filter(pmf.support, rv_ge_contains(rv, a))
    rv_ge_event(pmf, rv, a).contains(x) = (pmf.support.contains(x) and a <= rv(x))
}

/// On a support point, `a * 1_{X >= a}` is at most `X` when `X` is nonnegative there.
lemma markov_pointwise_a_indicator_le[T](pmf: DiscretePMF[T], rv: T -> Real, a: Real, t: T) {
    pmf.support.contains(t) and Real.0 <= rv(t) implies
        a * finite_event_indicator(rv_ge_event(pmf, rv, a), t) <= rv(t)
} by {
    if pmf.support.contains(t) and Real.0 <= rv(t) {
        if rv_ge_event(pmf, rv, a).contains(t) {
            finite_event_indicator_one_of_contains(rv_ge_event(pmf, rv, a), t)
            finite_event_indicator(rv_ge_event(pmf, rv, a), t) = Real.1
            a * Real.1 = a
            rv_ge_event_contains_eq(pmf, rv, a, t)
            rv_ge_event(pmf, rv, a).contains(t) = (pmf.support.contains(t) and a <= rv(t))
            a <= rv(t)
            a * finite_event_indicator(rv_ge_event(pmf, rv, a), t) <= rv(t)
        }
        if not rv_ge_event(pmf, rv, a).contains(t) {
            finite_event_indicator_zero_of_not_contains(rv_ge_event(pmf, rv, a), t)
            finite_event_indicator(rv_ge_event(pmf, rv, a), t) = Real.0
            a * Real.0 = Real.0
            Real.0 <= rv(t)
            a * finite_event_indicator(rv_ge_event(pmf, rv, a), t) <= rv(t)
        }
        rv_ge_event(pmf, rv, a).contains(t) or not rv_ge_event(pmf, rv, a).contains(t)
        a * finite_event_indicator(rv_ge_event(pmf, rv, a), t) <= rv(t)
    }
}

/// Markov's inequality: for a nonnegative random variable `X` and `a > 0`,
/// the probability that `X` is at least `a` is at most `E(X) / a`.
theorem markov_inequality[T](pmf: DiscretePMF[T], rv: T -> Real, a: Real) {
    (forall(t: T) { pmf.support.contains(t) implies Real.0 <= rv(t) }) and a > Real.0
        implies pmf_event_prob(pmf, rv_ge_event(pmf, rv, a)) <= discrete_expectation(pmf, rv) / a
} by {
    if forall(t: T) { pmf.support.contains(t) implies Real.0 <= rv(t) } and a > Real.0 {
        forall(t: T) {
            if pmf.support.contains(t) {
                markov_pointwise_a_indicator_le(pmf, rv, a, t)
                a * finite_event_indicator(rv_ge_event(pmf, rv, a), t) <= rv(t)
                mul_fn(a, finite_event_indicator(rv_ge_event(pmf, rv, a)), t) =
                    a * finite_event_indicator(rv_ge_event(pmf, rv, a), t)
                mul_fn(a, finite_event_indicator(rv_ge_event(pmf, rv, a)), t) <= rv(t)
            }
        }
        forall(t: T) {
            pmf.support.contains(t) implies mul_fn(a, finite_event_indicator(rv_ge_event(pmf, rv, a)), t) <= rv(t)
        }
        discrete_expectation_monotone(pmf, mul_fn(a, finite_event_indicator(rv_ge_event(pmf, rv, a))), rv)
        discrete_expectation(pmf, mul_fn(a, finite_event_indicator(rv_ge_event(pmf, rv, a)))) <= discrete_expectation(pmf, rv)
        discrete_expectation_scalar_mul(pmf, a, finite_event_indicator(rv_ge_event(pmf, rv, a)))
        discrete_expectation(pmf, mul_fn(a, finite_event_indicator(rv_ge_event(pmf, rv, a)))) =
            a * discrete_expectation(pmf, finite_event_indicator(rv_ge_event(pmf, rv, a)))
        discrete_expectation_indicator_eq_event_prob(pmf, rv_ge_event(pmf, rv, a))
        discrete_expectation(pmf, finite_event_indicator(rv_ge_event(pmf, rv, a))) =
            pmf_event_prob(pmf, rv_ge_event(pmf, rv, a))
        a * pmf_event_prob(pmf, rv_ge_event(pmf, rv, a)) <= discrete_expectation(pmf, rv)
        pmf_event_prob(pmf, rv_ge_event(pmf, rv, a)) * a <= discrete_expectation(pmf, rv)
        a > Real.0
        div_le_of_mul_le(pmf_event_prob(pmf, rv_ge_event(pmf, rv, a)), a, discrete_expectation(pmf, rv))
        pmf_event_prob(pmf, rv_ge_event(pmf, rv, a)) <= discrete_expectation(pmf, rv) / a
    }
}
