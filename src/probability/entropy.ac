from real import Real, log_some_of_pos_exists, log_one, log_mul,
    log_monotone, log_strict_monotone, mul_one_over, gt_zero_imp_pos, pos_gt_zero,
    mul_pos_pos, from_nat_real_pos_of_ne_zero, add_neg_eq_zero
from ordered_field import inverse_of_positive_is_positive, mul_le_mul_of_nonneg_right,
    zero_is_smaller_than_one
from algebra.add_ordered_group import add_le_add_right
from order import lt_imp_lte, not_gt_imp_lte, lte_antisymm, lt_of_lte_of_lt
from nat import Nat, from_nat
from finite_set import FiniteSet, fs_empty, fs_insert, finite_set_sum,
    finite_set_sum_empty, finite_set_sum_insert, finite_set_sum_add,
    finite_set_sum_scalar_mul, finite_set_sum_eq_list_sum, finite_set_has_unique_list,
    fs_from_list, finite_set_empty_contains_eq, finite_set_singleton_subset_of_contains
from data.basic.set import Set, list_set, list_set_contains_eq
from list import List, map, sum, sum_map_of_pointwise
from data.finite.finite_set_sum_real import finite_set_sum_nonneg
from data.finite.finite_set_product import finite_set_product
from data.finite.finite_set_sum_product import finite_set_sum_product_fubini_rows,
    finite_set_sum_eq_of_function_eq
from data.basic.functions import function_extensionality
from algebra.semigroup import mul_fn
from algebra.add_semigroup import add_fn
from probability.discrete_pmf import DiscretePMF, pmf_event_prob, pmf_event_prob_mono,
    pmf_event_prob_support, discrete_pmf_mass_nonneg, discrete_pmf_mass_zero_outside,
    const_real_fn
from probability.bernoulli_pmf import bernoulli_support, bernoulli_mass, bernoulli_mass_true,
    bernoulli_mass_false, bernoulli_support_sum, fs_empty_bool_no_false, fs_insert_false_no_true
from probability.discrete_uniform_pmf import finite_uniform_mass, finite_uniform_mass_of_contains,
    finite_set_sum_const_real_of_cardinality_is
from pair import Pair, pair_eta

/// The per-outcome entropy summand `p(x)·log p(x)`.  The convention
/// `0·log 0 = 0` is built in through `log_or_zero`, which defaults to zero
/// outside the positive reals.
define entropy_term[T](mass: T -> Real, x: T) -> Real {
    mass(x) * (mass(x)).log.get_or_else(Real.0)
}

/// The per-outcome entropy contribution `-p(x)·log p(x)`.
define entropy_surprise[T](mass: T -> Real, x: T) -> Real {
    -entropy_term(mass, x)
}

/// The Shannon entropy of a discrete PMF: `H(p) = -Σ_x p(x)·log p(x)`.
define discrete_entropy[T](pmf: DiscretePMF[T]) -> Real {
    finite_set_sum(pmf.support, entropy_surprise(pmf.mass))
}

/// The binary entropy function `h(p) = -p·log p - (1-p)·(1-p).log`.
define binary_entropy(p: Real) -> Real {
    -(p * p.log.get_or_else(Real.0) + (Real.1 - p) * (Real.1 - p).log.get_or_else(Real.0))
}

/// The reciprocal of a positive real is positive.
lemma entropy_real_one_div_pos(a: Real) {
    a > Real.0 implies Real.1 / a > Real.0
} by {
    if a > Real.0 {
        gt_zero_imp_pos(a)
        a.is_positive
        inverse_of_positive_is_positive(a)
        Real.0 < a.inverse
        a.inverse.is_positive
        Real.1.is_positive
        (Real.1 * a.inverse).is_positive
        Real.1 / a = Real.1 * a.inverse
        (Real.1 / a).is_positive
        Real.1 / a > Real.0
    }
}

/// The logarithm of one is zero.
lemma log_value_one {
    (Real.1).log.get_or_else(Real.0) = Real.0
} by {
    log_one
    Real.1.log = Option.some(Real.0)
    option_get_or_else_some[Real](Real.0, Real.0)
    option_get_or_else(Option.some(Real.0), Real.0) = Real.0
    (Real.1).log.get_or_else(Real.0) = Real.0
}

/// The logarithm of the reciprocal of a positive real is the negated logarithm.
lemma log_value_recip(x: Real) {
    x > Real.0 implies (Real.1 / x).log.get_or_else(Real.0) = -x.log.get_or_else(Real.0)
} by {
    if x > Real.0 {
        x != Real.0
        mul_one_over(x, x)
        x * (Real.1 / x) = x / x
        x / x = Real.1
        x * (Real.1 / x) = Real.1
        entropy_real_one_div_pos(x)
        Real.1 / x > Real.0
        log_some_of_pos_exists(x)
        let a: Real satisfy {
            x.log = Option.some(a)
        }
        log_some_of_pos_exists(Real.1 / x)
        let b: Real satisfy {
            (Real.1 / x).log = Option.some(b)
        }
        log_mul(x, Real.1 / x, a, b)
        x > Real.0 and Real.1 / x > Real.0 and x.log = Option.some(a) and (Real.1 / x).log = Option.some(b) implies (x * (Real.1 / x)).log = Option.some(a + b)
        (x * (Real.1 / x)).log = Option.some(a + b)
        x.log.get_or_else(Real.0) = option_get_or_else(x.log, Real.0)
        option_get_or_else_some[Real](a, Real.0)
        option_get_or_else(Option.some(a), Real.0) = a
        option_get_or_else_some[Real](a, Real.0)
        option_get_or_else(Option.some(a), Real.0) = a
        x.log.get_or_else(Real.0) = a
        (Real.1 / x).log.get_or_else(Real.0) = option_get_or_else((Real.1 / x).log, Real.0)
        option_get_or_else_some[Real](b, Real.0)
        option_get_or_else(Option.some(b), Real.0) = b
        (Real.1 / x).log.get_or_else(Real.0) = b
        a + b = x.log.get_or_else(Real.0) + (Real.1 / x).log.get_or_else(Real.0)
        (x * (Real.1 / x)).log = Option.some(x.log.get_or_else(Real.0) + (Real.1 / x).log.get_or_else(Real.0))
        Real.1.log = Option.some(x.log.get_or_else(Real.0) + (Real.1 / x).log.get_or_else(Real.0))
        log_one
        Option.some(Real.0) = Option.some(x.log.get_or_else(Real.0) + (Real.1 / x).log.get_or_else(Real.0))
        some_injective[Real](Real.0, x.log.get_or_else(Real.0) + (Real.1 / x).log.get_or_else(Real.0))
        Real.0 = x.log.get_or_else(Real.0) + (Real.1 / x).log.get_or_else(Real.0)
        x.log.get_or_else(Real.0) + (Real.1 / x).log.get_or_else(Real.0) = Real.0
        (x.log.get_or_else(Real.0) + (Real.1 / x).log.get_or_else(Real.0)) + -x.log.get_or_else(Real.0) = Real.0 + -x.log.get_or_else(Real.0)
        (Real.1 / x).log.get_or_else(Real.0) + (x.log.get_or_else(Real.0) + -x.log.get_or_else(Real.0)) = -x.log.get_or_else(Real.0)
        add_neg_eq_zero(x.log.get_or_else(Real.0))
        x.log.get_or_else(Real.0) + -x.log.get_or_else(Real.0) = Real.0
        (Real.1 / x).log.get_or_else(Real.0) + Real.0 = -x.log.get_or_else(Real.0)
        (Real.1 / x).log.get_or_else(Real.0) = -x.log.get_or_else(Real.0)
    }
}

/// The logarithm is nonpositive on positive reals at most one.
lemma log_value_le_zero_of_pos_le_one(x: Real) {
    x > Real.0 and x <= Real.1 implies x.log.get_or_else(Real.0) <= Real.0
} by {
    if x > Real.0 and x <= Real.1 {
        zero_is_smaller_than_one[Real]
        Real.1 > Real.0
        log_some_of_pos_exists(x)
        let a: Real satisfy {
            x.log = Option.some(a)
        }
        option_get_or_else_some[Real](a, Real.0)
        option_get_or_else(Option.some(a), Real.0) = a
        x.log.get_or_else(Real.0) = a
        x.log = Option.some(x.log.get_or_else(Real.0))
        log_one
        Real.1.log = Option.some(Real.0)
        log_monotone(x, Real.1, x.log.get_or_else(Real.0), Real.0)
        x.log.get_or_else(Real.0) <= Real.0
    }
}

/// The per-outcome entropy contribution `-p·log p` is nonnegative on `[0, 1]`.
lemma entropy_surprise_nonneg_of_unit(p: Real) {
    Real.0 <= p and p <= Real.1 implies Real.0 <= -(p * p.log.get_or_else(Real.0))
} by {
    if Real.0 <= p and p <= Real.1 {
        if p > Real.0 {
            log_value_le_zero_of_pos_le_one(p)
            p.log.get_or_else(Real.0) <= Real.0
            mul_le_mul_of_nonneg_right[Real](p.log.get_or_else(Real.0), Real.0, p)
            p.log.get_or_else(Real.0) * p <= Real.0 * p
            Real.0 * p = Real.0
            p.log.get_or_else(Real.0) * p <= Real.0
            p * p.log.get_or_else(Real.0) <= Real.0
            add_le_add_right[Real](p * p.log.get_or_else(Real.0), Real.0, -(p * p.log.get_or_else(Real.0)))
            p * p.log.get_or_else(Real.0) + -(p * p.log.get_or_else(Real.0)) <= Real.0 + -(p * p.log.get_or_else(Real.0))
            Real.0 <= -(p * p.log.get_or_else(Real.0))
        }
        if not p > Real.0 {
            not_gt_imp_lte(p, Real.0)
            p <= Real.0
            lte_antisymm[Real](p, Real.0)
            p = Real.0
            Real.0 * (Real.0).log.get_or_else(Real.0) = Real.0
            p * p.log.get_or_else(Real.0) = Real.0
            -(p * p.log.get_or_else(Real.0)) = Real.0
            Real.0 <= -(p * p.log.get_or_else(Real.0))
        }
        p > Real.0 or not p > Real.0
        Real.0 <= -(p * p.log.get_or_else(Real.0))
    }
}

/// Finite-set sums agree when the summands agree on the finite set.
lemma finite_set_sum_eq_of_contains_eq[T](s: FiniteSet[T], f: T -> Real, g: T -> Real) {
    (forall(x: T) { s.contains(x) implies f(x) = g(x) }) implies finite_set_sum(s, f) = finite_set_sum(s, g)
} by {
    if forall(x: T) { s.contains(x) implies f(x) = g(x) } {
        finite_set_has_unique_list(s)
        let items: List[T] satisfy {
            fs_from_list(items) = s and items.is_unique
        }
        forall(x: T) {
            if items.contains(x) {
                fs_from_list(items).underlying_set = list_set(items)
                list_set_contains_eq(items, x)
                list_set(items).contains(x) = items.contains(x)
                s.underlying_set.contains(x)
                s.contains(x)
                f(x) = g(x)
            }
        }
        forall(x: T) { items.contains(x) implies f(x) = g(x) }
        sum_map_of_pointwise[T, Real](items, f, g)
        sum[Real](map(items, f)) = sum[Real](map(items, g))
        finite_set_sum_eq_list_sum[T, Real](items, f)
        finite_set_sum_eq_list_sum[T, Real](items, g)
        finite_set_sum(s, f) = sum[Real](map(items, f))
        finite_set_sum(s, g) = sum[Real](map(items, g))
        finite_set_sum(s, f) = finite_set_sum(s, g)
    }
}

/// The sum over the singleton finite set `{x}` of `f` is `f(x)`.
lemma finite_set_sum_singleton_eq[T](x: T, f: T -> Real) {
    finite_set_sum(fs_insert(FiniteSet.empty[T], x), f) = f(x)
} by {
    let e0: FiniteSet[T] = FiniteSet.empty[T]
    finite_set_sum_empty[T, Real](f)
    finite_set_sum(fs_empty[T](Set[T].empty_set), f) = Real.0
    finite_set_empty_contains_eq[T](x)
    not e0.contains(x)
    finite_set_sum_insert[T, Real](e0, x, f)
    finite_set_sum(fs_insert(e0, x), f) = f(x) + finite_set_sum(e0, f)
    finite_set_sum(e0, f) = Real.0
    f(x) + Real.0 = f(x)
    finite_set_sum(fs_insert(e0, x), f) = f(x)
}

/// A point mass of a discrete PMF is at most one.
theorem pmf_mass_le_one[T](pmf: DiscretePMF[T], x: T) {
    pmf.support.contains(x) implies pmf.mass(x) <= Real.1
} by {
    if pmf.support.contains(x) {
        let pt = fs_insert(FiniteSet.empty[T], x)
        finite_set_singleton_subset_of_contains[T](pmf.support, x)
        pt.subset_eq(pmf.support)
        finite_set_sum_singleton_eq(x, pmf.mass)
        finite_set_sum(pt, pmf.mass) = pmf.mass(x)
        pmf_event_prob(pmf, pt) = finite_set_sum(pt, pmf.mass)
        pmf_event_prob(pmf, pt) = pmf.mass(x)
        pmf_event_prob_mono[T](pmf, pt, pmf.support)
        pmf_event_prob(pmf, pt) <= pmf_event_prob(pmf, pmf.support)
        pmf_event_prob_support[T](pmf)
        pmf_event_prob(pmf, pmf.support) = Real.1
        pmf.mass(x) <= Real.1
    }
}

/// The entropy of a uniform distribution over `n` outcomes is `log n`.
theorem uniform_entropy_eq_log_cardinality[T](support: FiniteSet[T], n: Nat, pmf: DiscretePMF[T]) {
    pmf.support = support and support.cardinality_is(n) and from_nat[Real](n) > Real.0 and
    (forall(x: T) { support.contains(x) implies pmf.mass(x) = finite_uniform_mass(support, n, x) })
    implies discrete_entropy(pmf) = (from_nat[Real](n)).log.get_or_else(Real.0)
} by {
    if pmf.support = support and support.cardinality_is(n) and from_nat[Real](n) > Real.0 and
        (forall(x: T) { support.contains(x) implies pmf.mass(x) = finite_uniform_mass(support, n, x) }) {
        let c = Real.1 / from_nat[Real](n)
        let lc = c.log.get_or_else(Real.0)
        let uniform = finite_uniform_mass(support, n)

        forall(x: T) {
            if support.contains(x) {
                finite_uniform_mass_of_contains(support, n, x)
                finite_uniform_mass(support, n, x) = c
                pmf.mass(x) = finite_uniform_mass(support, n, x)
                pmf.mass(x) = c
                entropy_surprise(pmf.mass, x) = -(pmf.mass(x) * (pmf.mass(x)).log.get_or_else(Real.0))
                entropy_surprise(pmf.mass, x) = -(c * c.log.get_or_else(Real.0))
                entropy_surprise(uniform, x) = -(uniform(x) * (uniform(x)).log.get_or_else(Real.0))
                uniform(x) = c
                entropy_surprise(uniform, x) = -(c * c.log.get_or_else(Real.0))
                entropy_surprise(pmf.mass, x) = entropy_surprise(uniform, x)
            }
        }
        forall(x: T) { support.contains(x) implies entropy_surprise(pmf.mass, x) = entropy_surprise(uniform, x) }
        finite_set_sum_eq_of_contains_eq(support, entropy_surprise(pmf.mass), entropy_surprise(uniform))
        finite_set_sum(support, entropy_surprise(pmf.mass)) = finite_set_sum(support, entropy_surprise(uniform))

        forall(x: T) {
            if support.contains(x) {
                finite_uniform_mass_of_contains(support, n, x)
                finite_uniform_mass(support, n, x) = c
                entropy_surprise(uniform, x) = -(uniform(x) * (uniform(x)).log.get_or_else(Real.0))
                uniform(x) = c
                entropy_surprise(uniform, x) = -(c * c.log.get_or_else(Real.0))
                lc = c.log.get_or_else(Real.0)
                entropy_surprise(uniform, x) = -(c * lc)
                const_real_fn[T](-(c * lc), x) = -(c * lc)
                entropy_surprise(uniform, x) = const_real_fn[T](-(c * lc), x)
            }
        }
        forall(x: T) {
            support.contains(x) implies entropy_surprise(uniform, x) = const_real_fn[T](-(c * lc), x)
        }
        finite_set_sum_eq_of_contains_eq(support, entropy_surprise(uniform), const_real_fn[T](-(c * lc)))
        finite_set_sum(support, entropy_surprise(uniform)) = finite_set_sum(support, const_real_fn[T](-(c * lc)))

        finite_set_sum_const_real_of_cardinality_is(support, -(c * lc), n)
        finite_set_sum(support, const_real_fn[T](-(c * lc))) = from_nat[Real](n) * (-(c * lc))
        from_nat[Real](n) * (-(c * lc)) = -(from_nat[Real](n) * (c * lc))

        from_nat[Real](n) != Real.0
        mul_one_over(from_nat[Real](n), from_nat[Real](n))
        from_nat[Real](n) * (Real.1 / from_nat[Real](n)) = from_nat[Real](n) / from_nat[Real](n)
        from_nat[Real](n) / from_nat[Real](n) = Real.1
        from_nat[Real](n) * c = Real.1
        from_nat[Real](n) * (c * lc) = (from_nat[Real](n) * c) * lc
        from_nat[Real](n) * (c * lc) = Real.1 * lc
        Real.1 * lc = lc
        from_nat[Real](n) * (c * lc) = lc
        from_nat[Real](n) * (-(c * lc)) = -lc

        finite_set_sum(support, entropy_surprise(uniform)) = -lc
        finite_set_sum(support, entropy_surprise(pmf.mass)) = -lc
        discrete_entropy(pmf) = finite_set_sum(pmf.support, entropy_surprise(pmf.mass))
        discrete_entropy(pmf) = finite_set_sum(support, entropy_surprise(pmf.mass))
        discrete_entropy(pmf) = -lc

        log_value_recip(from_nat[Real](n))
        (Real.1 / from_nat[Real](n)).log.get_or_else(Real.0) = -(from_nat[Real](n)).log.get_or_else(Real.0)
        lc = -(from_nat[Real](n)).log.get_or_else(Real.0)
        -lc = (from_nat[Real](n)).log.get_or_else(Real.0)
        discrete_entropy(pmf) = (from_nat[Real](n)).log.get_or_else(Real.0)
    }
}

/// The logarithm of `n` is the entropy of the uniform distribution over `n` outcomes.
theorem uniform_entropy_log_eq_some[T](support: FiniteSet[T], n: Nat, pmf: DiscretePMF[T]) {
    pmf.support = support and support.cardinality_is(n) and from_nat[Real](n) > Real.0 and
    (forall(x: T) { support.contains(x) implies pmf.mass(x) = finite_uniform_mass(support, n, x) })
    implies (from_nat[Real](n)).log = Option.some(discrete_entropy(pmf))
} by {
    if pmf.support = support and support.cardinality_is(n) and from_nat[Real](n) > Real.0 and
        (forall(x: T) { support.contains(x) implies pmf.mass(x) = finite_uniform_mass(support, n, x) }) {
        uniform_entropy_eq_log_cardinality(support, n, pmf)
        discrete_entropy(pmf) = (from_nat[Real](n)).log.get_or_else(Real.0)
        log_some_of_pos_exists(from_nat[Real](n))
        let a: Real satisfy {
            from_nat[Real](n).log = Option.some(a)
        }
        option_get_or_else_some[Real](a, Real.0)
        option_get_or_else(Option.some(a), Real.0) = a
        (from_nat[Real](n)).log.get_or_else(Real.0) = a
        (from_nat[Real](n)).log = Option.some((from_nat[Real](n)).log.get_or_else(Real.0))
        (from_nat[Real](n)).log = Option.some(discrete_entropy(pmf))
    }
}

/// The sum of the entropy summands over the Bernoulli support.
theorem bernoulli_entropy_term_sum(p: Real) {
    finite_set_sum(bernoulli_support, entropy_term(bernoulli_mass(p)))
        = p * p.log.get_or_else(Real.0) + (Real.1 - p) * (Real.1 - p).log.get_or_else(Real.0)
} by {
    let e0: FiniteSet[Bool] = fs_empty[Bool](Set[Bool].empty_set)
    let s1: FiniteSet[Bool] = fs_insert(e0, false)
    let f: Bool -> Real = entropy_term(bernoulli_mass(p))
    finite_set_sum_empty[Bool, Real](f)
    fs_empty_bool_no_false
    finite_set_sum_insert[Bool, Real](e0, false, f)
    finite_set_sum(s1, f) = f(false) + finite_set_sum(e0, f)
    finite_set_sum(s1, f) = f(false) + Real.0
    fs_insert_false_no_true
    finite_set_sum_insert[Bool, Real](s1, true, f)
    finite_set_sum(fs_insert(s1, true), f) = f(true) + finite_set_sum(s1, f)
    bernoulli_support = fs_insert(s1, true)
    finite_set_sum(bernoulli_support, f) = f(true) + (f(false) + Real.0)
    entropy_term(bernoulli_mass(p), true) = bernoulli_mass(p, true) * (bernoulli_mass(p, true)).log.get_or_else(Real.0)
    bernoulli_mass_true(p)
    bernoulli_mass(p, true) = p
    f(true) = p * p.log.get_or_else(Real.0)
    entropy_term(bernoulli_mass(p), false) = bernoulli_mass(p, false) * (bernoulli_mass(p, false)).log.get_or_else(Real.0)
    bernoulli_mass_false(p)
    bernoulli_mass(p, false) = Real.1 - p
    f(false) = (Real.1 - p) * (Real.1 - p).log.get_or_else(Real.0)
    finite_set_sum(bernoulli_support, f) = p * p.log.get_or_else(Real.0) + ((Real.1 - p) * (Real.1 - p).log.get_or_else(Real.0) + Real.0)
    p * p.log.get_or_else(Real.0) + ((Real.1 - p) * (Real.1 - p).log.get_or_else(Real.0) + Real.0) =
        p * p.log.get_or_else(Real.0) + (Real.1 - p) * (Real.1 - p).log.get_or_else(Real.0)
    finite_set_sum(bernoulli_support, f) = p * p.log.get_or_else(Real.0) + (Real.1 - p) * (Real.1 - p).log.get_or_else(Real.0)
}

/// The entropy of a Bernoulli distribution is the binary entropy function.
theorem bernoulli_entropy_eq_binary_entropy(p: Real, pmf: DiscretePMF[Bool]) {
    pmf.support = bernoulli_support and
    (forall(b: Bool) { pmf.mass(b) = bernoulli_mass(p, b) })
    implies discrete_entropy(pmf) = binary_entropy(p)
} by {
    if pmf.support = bernoulli_support and (forall(b: Bool) { pmf.mass(b) = bernoulli_mass(p, b) }) {
        let f: Bool -> Real = entropy_surprise(pmf.mass)
        discrete_entropy(pmf) = finite_set_sum(pmf.support, entropy_surprise(pmf.mass))
        discrete_entropy(pmf) = finite_set_sum(bernoulli_support, f)

        pmf.mass(true) = bernoulli_mass(p, true)
        bernoulli_mass_true(p)
        bernoulli_mass(p, true) = p
        pmf.mass(true) = p
        f(true) = -(pmf.mass(true) * (pmf.mass(true)).log.get_or_else(Real.0))
        f(true) = -(p * p.log.get_or_else(Real.0))

        pmf.mass(false) = bernoulli_mass(p, false)
        bernoulli_mass_false(p)
        bernoulli_mass(p, false) = Real.1 - p
        pmf.mass(false) = Real.1 - p
        f(false) = -(pmf.mass(false) * (pmf.mass(false)).log.get_or_else(Real.0))
        f(false) = -((Real.1 - p) * (Real.1 - p).log.get_or_else(Real.0))

        let e0: FiniteSet[Bool] = fs_empty[Bool](Set[Bool].empty_set)
        let s1: FiniteSet[Bool] = fs_insert(e0, false)
        finite_set_sum_empty[Bool, Real](f)
        fs_empty_bool_no_false
        finite_set_sum_insert[Bool, Real](e0, false, f)
        finite_set_sum(s1, f) = f(false) + finite_set_sum(e0, f)
        finite_set_sum(s1, f) = f(false) + Real.0
        fs_insert_false_no_true
        finite_set_sum_insert[Bool, Real](s1, true, f)
        finite_set_sum(fs_insert(s1, true), f) = f(true) + finite_set_sum(s1, f)
        bernoulli_support = fs_insert(s1, true)
        finite_set_sum(bernoulli_support, f) = f(true) + (f(false) + Real.0)
        finite_set_sum(bernoulli_support, f) = -(p * p.log.get_or_else(Real.0)) + (-((Real.1 - p) * (Real.1 - p).log.get_or_else(Real.0)) + Real.0)
        -(p * p.log.get_or_else(Real.0)) + (-((Real.1 - p) * (Real.1 - p).log.get_or_else(Real.0)) + Real.0) =
            -(p * p.log.get_or_else(Real.0) + (Real.1 - p) * (Real.1 - p).log.get_or_else(Real.0))
        finite_set_sum(bernoulli_support, f) = -(p * p.log.get_or_else(Real.0) + (Real.1 - p) * (Real.1 - p).log.get_or_else(Real.0))
        discrete_entropy(pmf) = -(p * p.log.get_or_else(Real.0) + (Real.1 - p) * (Real.1 - p).log.get_or_else(Real.0))
        binary_entropy(p) = -(p * p.log.get_or_else(Real.0) + (Real.1 - p) * (Real.1 - p).log.get_or_else(Real.0))
        discrete_entropy(pmf) = binary_entropy(p)
    }
}

/// The Shannon entropy of a discrete PMF is nonnegative.
theorem discrete_entropy_nonneg[T](pmf: DiscretePMF[T]) {
    Real.0 <= discrete_entropy(pmf)
} by {
    let surprise = entropy_surprise(pmf.mass)
    forall(x: T) {
        discrete_pmf_mass_nonneg(pmf, x)
        Real.0 <= pmf.mass(x)
        if pmf.support.contains(x) {
            pmf_mass_le_one(pmf, x)
            pmf.mass(x) <= Real.1
            entropy_surprise_nonneg_of_unit(pmf.mass(x))
            Real.0 <= -(pmf.mass(x) * (pmf.mass(x)).log.get_or_else(Real.0))
            entropy_surprise(pmf.mass, x) = -(pmf.mass(x) * (pmf.mass(x)).log.get_or_else(Real.0))
            Real.0 <= entropy_surprise(pmf.mass, x)
            surprise(x) = entropy_surprise(pmf.mass, x)
            Real.0 <= surprise(x)
        }
        if not pmf.support.contains(x) {
            discrete_pmf_mass_zero_outside(pmf, x)
            pmf.mass(x) = Real.0
            Real.0 * (Real.0).log.get_or_else(Real.0) = Real.0
            pmf.mass(x) * (pmf.mass(x)).log.get_or_else(Real.0) = Real.0
            -(pmf.mass(x) * (pmf.mass(x)).log.get_or_else(Real.0)) = Real.0
            Real.0 <= -(pmf.mass(x) * (pmf.mass(x)).log.get_or_else(Real.0))
            entropy_surprise(pmf.mass, x) = -(pmf.mass(x) * (pmf.mass(x)).log.get_or_else(Real.0))
            Real.0 <= entropy_surprise(pmf.mass, x)
            surprise(x) = entropy_surprise(pmf.mass, x)
            Real.0 <= surprise(x)
        }
        pmf.support.contains(x) or not pmf.support.contains(x)
        Real.0 <= surprise(x)
    }
    finite_set_sum_nonneg[T](pmf.support, surprise)
    Real.0 <= finite_set_sum(pmf.support, surprise)
    discrete_entropy(pmf) = finite_set_sum(pmf.support, surprise)
    Real.0 <= discrete_entropy(pmf)
}

/// The binary entropy of a deterministic outcome is zero.
theorem binary_entropy_zero {
    binary_entropy(Real.0) = Real.0
} by {
    binary_entropy(Real.0) = -(Real.0 * (Real.0).log.get_or_else(Real.0) + (Real.1 - Real.0) * (Real.1 - Real.0).log.get_or_else(Real.0))
    Real.0 * (Real.0).log.get_or_else(Real.0) = Real.0
    Real.1 - Real.0 = Real.1
    log_value_one
    (Real.1).log.get_or_else(Real.0) = Real.0
    Real.1 * (Real.1).log.get_or_else(Real.0) = Real.0
    binary_entropy(Real.0) = -(Real.0 + Real.0)
    Real.0 + Real.0 = Real.0
    binary_entropy(Real.0) = -Real.0
    -Real.0 = Real.0
    binary_entropy(Real.0) = Real.0
}

/// The binary entropy of a deterministic outcome is zero.
theorem binary_entropy_one {
    binary_entropy(Real.1) = Real.0
} by {
    binary_entropy(Real.1) = -(Real.1 * (Real.1).log.get_or_else(Real.0) + (Real.1 - Real.1) * (Real.1 - Real.1).log.get_or_else(Real.0))
    log_value_one
    (Real.1).log.get_or_else(Real.0) = Real.0
    Real.1 * Real.0 = Real.0
    Real.1 - Real.1 = Real.0
    Real.0 * (Real.0).log.get_or_else(Real.0) = Real.0
    binary_entropy(Real.1) = -(Real.0 + Real.0)
    Real.0 + Real.0 = Real.0
    binary_entropy(Real.1) = -Real.0
    -Real.0 = Real.0
    binary_entropy(Real.1) = Real.0
}

/// The binary entropy at one half is `log 2`.
theorem binary_entropy_half {
    binary_entropy(Real.1 / (Real.1 + Real.1)) = (Real.1 + Real.1).log.get_or_else(Real.0)
} by {
    let p = Real.1 / (Real.1 + Real.1)
    let two = Real.1 + Real.1

    zero_is_smaller_than_one[Real]
    Real.0 < Real.1
    lt_imp_lte[Real](Real.0, Real.1)
    Real.0 <= Real.1
    add_le_add_right[Real](Real.0, Real.1, Real.1)
    Real.0 + Real.1 <= Real.1 + Real.1
    Real.0 + Real.1 = Real.1
    Real.1 <= two
    lt_of_lte_of_lt[Real](Real.0, Real.1, two)
    Real.0 < two
    two > Real.0

    two != Real.0
    mul_one_over(two, two)
    two * (Real.1 / two) = two / two
    two / two = Real.1
    two * p = Real.1
    p + p = two * p
    p + p = Real.1
    Real.1 - p = p

    binary_entropy(p) = -(p * p.log.get_or_else(Real.0) + (Real.1 - p) * (Real.1 - p).log.get_or_else(Real.0))
    binary_entropy(p) = -(p * p.log.get_or_else(Real.0) + p * p.log.get_or_else(Real.0))
    -(p * p.log.get_or_else(Real.0) + p * p.log.get_or_else(Real.0)) = -((p + p) * p.log.get_or_else(Real.0))
    -((p + p) * p.log.get_or_else(Real.0)) = -p.log.get_or_else(Real.0)
    binary_entropy(p) = -p.log.get_or_else(Real.0)

    log_value_recip(two)
    (Real.1 / two).log.get_or_else(Real.0) = -two.log.get_or_else(Real.0)
    p.log.get_or_else(Real.0) = -two.log.get_or_else(Real.0)
    binary_entropy(p) = -(-two.log.get_or_else(Real.0))
    -(-two.log.get_or_else(Real.0)) = two.log.get_or_else(Real.0)
    binary_entropy(p) = two.log.get_or_else(Real.0)
    binary_entropy(Real.1 / (Real.1 + Real.1)) = (Real.1 + Real.1).log.get_or_else(Real.0)
}

// ---------------------------------------------------------------------------
// Maximum entropy over two outcomes.
//
// The classical statement is that the uniform distribution maximizes entropy:
// for every `0 <= p <= 1`,
//
//     binary_entropy(p) <= log 2.
//
// Equivalently, `p·(1/p).log + (1-p)·(1/(1-p)).log <= log 2`.  The standard
// proof applies the tangent-line inequality `log x <= x - 1` to
// `x = 1/(2p)` and `x = 1/(2(1-p))`, obtaining
//
//     p·(1/(2p)).log <= 1/2 - p          and          (1-p)·(1/(2(1-p))).log <= p - 1/2,
//
// and adds the two inequalities, using `(x·y).log = log x + log y` to split
// `(1/(2p)).log = (1/p).log - log 2`.  The inequality `log x <= x - 1` is
// proved in `src/real/log_inequalities.ac` (`log_le_sub_one`) and ultimately
// rests on `exp x >= 1 + x` (`src/real/exp_inequalities.ac`), neither of
// which is re-exported through the `real` package interface, so the proof is
// left out here.  The extreme values are proved above:
// `binary_entropy(0) = binary_entropy(1) = 0` and, at the maximizer,
// `binary_entropy(1/2) = log 2`.
// ---------------------------------------------------------------------------

/// The mass of the product of two Bernoulli distributions on a pair of Booleans.
define bernoulli_pair_mass(p: Real, q: Real, pair: Pair[Bool, Bool]) -> Real {
    bernoulli_mass(p, pair.first) * bernoulli_mass(q, pair.second)
}

/// The `p`-row of the product entropy summand: `m1·log m1·m2`.
define bernoulli_row1(p: Real, q: Real, x: Bool, y: Bool) -> Real {
    bernoulli_mass(p, x) * (bernoulli_mass(p, x)).log.get_or_else(Real.0) * bernoulli_mass(q, y)
}

/// The `q`-row of the product entropy summand: `m2·log m2·m1`.
define bernoulli_row2(p: Real, q: Real, x: Bool, y: Bool) -> Real {
    bernoulli_mass(q, y) * (bernoulli_mass(q, y)).log.get_or_else(Real.0) * bernoulli_mass(p, x)
}

/// The first row as a function on pairs.
define bernoulli_row1_fn(p: Real, q: Real, pair: Pair[Bool, Bool]) -> Real {
    bernoulli_row1(p, q, pair.first, pair.second)
}

/// The second row as a function on pairs.
define bernoulli_row2_fn(p: Real, q: Real, pair: Pair[Bool, Bool]) -> Real {
    bernoulli_row2(p, q, pair.first, pair.second)
}

/// The Bernoulli masses are positive on the open unit interval.
lemma bernoulli_mass_pos_open(p: Real, b: Bool) {
    Real.0 < p and p < Real.1 implies bernoulli_mass(p, b) > Real.0
} by {
    if Real.0 < p and p < Real.1 {
        if b {
            bernoulli_mass_true(p)
            bernoulli_mass(p, b) = p
            bernoulli_mass(p, b) > Real.0
        }
        if not b {
            bernoulli_mass_false(p)
            bernoulli_mass(p, b) = Real.1 - p
            p < Real.1
            p + -p < Real.1 + -p
            Real.0 < Real.1 - p
            bernoulli_mass(p, b) > Real.0
        }
        b or not b
        bernoulli_mass(p, b) > Real.0
    }
}

/// The entropy summand of a product of positive masses decomposes into the two
/// per-factor contributions.
lemma product_entropy_term_decomp(m1: Real, m2: Real) {
    m1 > Real.0 and m2 > Real.0 implies
    (m1 * m2) * (m1 * m2).log.get_or_else(Real.0) =
        m1 * m1.log.get_or_else(Real.0) * m2 + m2 * m2.log.get_or_else(Real.0) * m1
} by {
    if m1 > Real.0 and m2 > Real.0 {
        log_some_of_pos_exists(m1)
        let a: Real satisfy {
            m1.log = Option.some(a)
        }
        log_some_of_pos_exists(m2)
        let b: Real satisfy {
            m2.log = Option.some(b)
        }
        log_mul(m1, m2, a, b)
        m1 > Real.0 and m2 > Real.0 and m1.log = Option.some(a) and m2.log = Option.some(b) implies (m1 * m2).log = Option.some(a + b)
        (m1 * m2).log = Option.some(a + b)
        m1.log.get_or_else(Real.0) = option_get_or_else(m1.log, Real.0)
        option_get_or_else_some[Real](a, Real.0)
        option_get_or_else(Option.some(a), Real.0) = a
        option_get_or_else_some[Real](a, Real.0)
        option_get_or_else(Option.some(a), Real.0) = a
        m1.log.get_or_else(Real.0) = a
        m2.log.get_or_else(Real.0) = option_get_or_else(m2.log, Real.0)
        option_get_or_else_some[Real](b, Real.0)
        option_get_or_else(Option.some(b), Real.0) = b
        option_get_or_else_some[Real](b, Real.0)
        option_get_or_else(Option.some(b), Real.0) = b
        m2.log.get_or_else(Real.0) = b
        (m1 * m2).log.get_or_else(Real.0) = option_get_or_else((m1 * m2).log, Real.0)
        option_get_or_else_some[Real](a + b, Real.0)
        option_get_or_else(Option.some(a + b), Real.0) = a + b
        (m1 * m2).log.get_or_else(Real.0) = a + b
        (m1 * m2).log.get_or_else(Real.0) = m1.log.get_or_else(Real.0) + m2.log.get_or_else(Real.0)
        (m1 * m2) * (m1 * m2).log.get_or_else(Real.0) = (m1 * m2) * (m1.log.get_or_else(Real.0) + m2.log.get_or_else(Real.0))
        (m1 * m2) * (m1.log.get_or_else(Real.0) + m2.log.get_or_else(Real.0)) =
            m1 * m2 * m1.log.get_or_else(Real.0) + m1 * m2 * m2.log.get_or_else(Real.0)
        m1 * m2 * m1.log.get_or_else(Real.0) = (m1 * m2) * m1.log.get_or_else(Real.0)
        m1 * m2 = m2 * m1
        (m1 * m2) * m1.log.get_or_else(Real.0) = (m2 * m1) * m1.log.get_or_else(Real.0)
        (m2 * m1) * m1.log.get_or_else(Real.0) = m2 * (m1 * m1.log.get_or_else(Real.0))
        m2 * (m1 * m1.log.get_or_else(Real.0)) = (m1 * m1.log.get_or_else(Real.0)) * m2
        (m1 * m1.log.get_or_else(Real.0)) * m2 = m1 * m1.log.get_or_else(Real.0) * m2
        m1 * m2 * m2.log.get_or_else(Real.0) = (m1 * m2) * m2.log.get_or_else(Real.0)
        (m1 * m2) * m2.log.get_or_else(Real.0) = (m2 * m1) * m2.log.get_or_else(Real.0)
        (m2 * m1) * m2.log.get_or_else(Real.0) = m2 * (m1 * m2.log.get_or_else(Real.0))
        m1 * m2.log.get_or_else(Real.0) = m2.log.get_or_else(Real.0) * m1
        m2 * (m1 * m2.log.get_or_else(Real.0)) = m2 * (m2.log.get_or_else(Real.0) * m1)
        m2 * (m2.log.get_or_else(Real.0) * m1) = (m2 * m2.log.get_or_else(Real.0)) * m1
        (m2 * m2.log.get_or_else(Real.0)) * m1 = m2 * m2.log.get_or_else(Real.0) * m1
        m1 * m2 * m1.log.get_or_else(Real.0) + m1 * m2 * m2.log.get_or_else(Real.0) =
            m1 * m1.log.get_or_else(Real.0) * m2 + m2 * m2.log.get_or_else(Real.0) * m1
        (m1 * m2) * (m1 * m2).log.get_or_else(Real.0) =
            m1 * m1.log.get_or_else(Real.0) * m2 + m2 * m2.log.get_or_else(Real.0) * m1
    }
}

/// The product entropy summand splits into the two row contributions.
lemma bernoulli_pair_entropy_term_split(p: Real, q: Real, pair: Pair[Bool, Bool]) {
    Real.0 < p and p < Real.1 and Real.0 < q and q < Real.1 implies
    entropy_term(bernoulli_pair_mass(p, q), pair)
        = bernoulli_row1_fn(p, q, pair) + bernoulli_row2_fn(p, q, pair)
} by {
    if Real.0 < p and p < Real.1 and Real.0 < q and q < Real.1 {
        bernoulli_mass_pos_open(p, pair.first)
        bernoulli_mass(p, pair.first) > Real.0
        bernoulli_mass_pos_open(q, pair.second)
        bernoulli_mass(q, pair.second) > Real.0
        entropy_term(bernoulli_pair_mass(p, q), pair) =
            bernoulli_pair_mass(p, q, pair) * (bernoulli_pair_mass(p, q, pair)).log.get_or_else(Real.0)
        bernoulli_pair_mass(p, q, pair) = bernoulli_mass(p, pair.first) * bernoulli_mass(q, pair.second)
        entropy_term(bernoulli_pair_mass(p, q), pair) =
            (bernoulli_mass(p, pair.first) * bernoulli_mass(q, pair.second)) *
                (bernoulli_mass(p, pair.first) * bernoulli_mass(q, pair.second)).log.get_or_else(Real.0)
        product_entropy_term_decomp(bernoulli_mass(p, pair.first), bernoulli_mass(q, pair.second))
        (bernoulli_mass(p, pair.first) * bernoulli_mass(q, pair.second)) * (bernoulli_mass(p, pair.first) * bernoulli_mass(q, pair.second)).log.get_or_else(Real.0) = bernoulli_mass(p, pair.first) * (bernoulli_mass(p, pair.first)).log.get_or_else(Real.0) * bernoulli_mass(q, pair.second) + bernoulli_mass(q, pair.second) * (bernoulli_mass(q, pair.second)).log.get_or_else(Real.0) * bernoulli_mass(p, pair.first)
        entropy_term(bernoulli_pair_mass(p, q), pair) = bernoulli_mass(p, pair.first) * (bernoulli_mass(p, pair.first)).log.get_or_else(Real.0) * bernoulli_mass(q, pair.second) + bernoulli_mass(q, pair.second) * (bernoulli_mass(q, pair.second)).log.get_or_else(Real.0) * bernoulli_mass(p, pair.first)
        bernoulli_row1_fn(p, q, pair) = bernoulli_row1(p, q, pair.first, pair.second)
        bernoulli_row1(p, q, pair.first, pair.second) = bernoulli_mass(p, pair.first) * (bernoulli_mass(p, pair.first)).log.get_or_else(Real.0) * bernoulli_mass(q, pair.second)
        bernoulli_row1_fn(p, q, pair) = bernoulli_mass(p, pair.first) * (bernoulli_mass(p, pair.first)).log.get_or_else(Real.0) * bernoulli_mass(q, pair.second)
        bernoulli_row2_fn(p, q, pair) = bernoulli_row2(p, q, pair.first, pair.second)
        bernoulli_row2(p, q, pair.first, pair.second) = bernoulli_mass(q, pair.second) * (bernoulli_mass(q, pair.second)).log.get_or_else(Real.0) * bernoulli_mass(p, pair.first)
        bernoulli_row2_fn(p, q, pair) = bernoulli_mass(q, pair.second) * (bernoulli_mass(q, pair.second)).log.get_or_else(Real.0) * bernoulli_mass(p, pair.first)
        bernoulli_row1_fn(p, q, pair) + bernoulli_row2_fn(p, q, pair) = bernoulli_mass(p, pair.first) * (bernoulli_mass(p, pair.first)).log.get_or_else(Real.0) * bernoulli_mass(q, pair.second) + bernoulli_mass(q, pair.second) * (bernoulli_mass(q, pair.second)).log.get_or_else(Real.0) * bernoulli_mass(p, pair.first)
        entropy_term(bernoulli_pair_mass(p, q), pair) =
            bernoulli_row1_fn(p, q, pair) + bernoulli_row2_fn(p, q, pair)
    }
}

/// The sum of the first row over the product support.
theorem bernoulli_product_row1_total(p: Real, q: Real) {
    finite_set_sum(finite_set_product(bernoulli_support, bernoulli_support), bernoulli_row1_fn(p, q))
        = p * p.log.get_or_else(Real.0) + (Real.1 - p) * (Real.1 - p).log.get_or_else(Real.0)
} by {
    finite_set_sum_product_fubini_rows[Bool, Bool, Real](
        bernoulli_support, bernoulli_support, bernoulli_row1_fn(p, q))
    finite_set_sum(finite_set_product(bernoulli_support, bernoulli_support), bernoulli_row1_fn(p, q)) =
        finite_set_sum(bernoulli_support, function(x: Bool) {
            finite_set_sum(bernoulli_support, function(y: Bool) { bernoulli_row1_fn(p, q, Pair.new(x, y)) })
        })
    forall(x: Bool) {
        forall(y: Bool) {
            bernoulli_row1_fn(p, q, Pair.new(x, y)) = bernoulli_row1(p, q, x, y)
            bernoulli_row1(p, q, x, y) =
                bernoulli_mass(p, x) * (bernoulli_mass(p, x)).log.get_or_else(Real.0) * bernoulli_mass(q, y)
            mul_fn(bernoulli_mass(p, x) * (bernoulli_mass(p, x)).log.get_or_else(Real.0), bernoulli_mass(q), y) =
                (bernoulli_mass(p, x) * (bernoulli_mass(p, x)).log.get_or_else(Real.0)) * bernoulli_mass(q, y)
            bernoulli_row1_fn(p, q, Pair.new(x, y)) =
                mul_fn(bernoulli_mass(p, x) * (bernoulli_mass(p, x)).log.get_or_else(Real.0), bernoulli_mass(q), y)
        }
        function_extensionality[Bool, Real](
            function(y: Bool) { bernoulli_row1_fn(p, q, Pair.new(x, y)) },
            mul_fn(bernoulli_mass(p, x) * (bernoulli_mass(p, x)).log.get_or_else(Real.0), bernoulli_mass(q)))
        finite_set_sum_eq_of_function_eq[Bool, Real](bernoulli_support,
            function(y: Bool) { bernoulli_row1_fn(p, q, Pair.new(x, y)) },
            mul_fn(bernoulli_mass(p, x) * (bernoulli_mass(p, x)).log.get_or_else(Real.0), bernoulli_mass(q)))
        finite_set_sum(bernoulli_support, function(y: Bool) { bernoulli_row1_fn(p, q, Pair.new(x, y)) }) =
            finite_set_sum(bernoulli_support,
                mul_fn(bernoulli_mass(p, x) * (bernoulli_mass(p, x)).log.get_or_else(Real.0), bernoulli_mass(q)))
        finite_set_sum_scalar_mul[Bool, Real](
            bernoulli_mass(p, x) * (bernoulli_mass(p, x)).log.get_or_else(Real.0), bernoulli_support, bernoulli_mass(q))
        (bernoulli_mass(p, x) * (bernoulli_mass(p, x)).log.get_or_else(Real.0)) *
            finite_set_sum(bernoulli_support, bernoulli_mass(q)) =
            finite_set_sum(bernoulli_support,
                mul_fn(bernoulli_mass(p, x) * (bernoulli_mass(p, x)).log.get_or_else(Real.0), bernoulli_mass(q)))
        bernoulli_support_sum(q)
        finite_set_sum(bernoulli_support, bernoulli_mass(q)) = Real.1
        (bernoulli_mass(p, x) * (bernoulli_mass(p, x)).log.get_or_else(Real.0)) * Real.1 =
            bernoulli_mass(p, x) * (bernoulli_mass(p, x)).log.get_or_else(Real.0)
        finite_set_sum(bernoulli_support, function(y: Bool) { bernoulli_row1_fn(p, q, Pair.new(x, y)) }) =
            bernoulli_mass(p, x) * (bernoulli_mass(p, x)).log.get_or_else(Real.0)
    }
    forall(x: Bool) {
        finite_set_sum(bernoulli_support, function(y: Bool) { bernoulli_row1_fn(p, q, Pair.new(x, y)) }) =
            bernoulli_mass(p, x) * (bernoulli_mass(p, x)).log.get_or_else(Real.0)
        entropy_term(bernoulli_mass(p), x) = bernoulli_mass(p, x) * (bernoulli_mass(p, x)).log.get_or_else(Real.0)
        finite_set_sum(bernoulli_support, function(y: Bool) { bernoulli_row1_fn(p, q, Pair.new(x, y)) }) =
            entropy_term(bernoulli_mass(p), x)
    }
    function_extensionality[Bool, Real](
        function(x: Bool) {
            finite_set_sum(bernoulli_support, function(y: Bool) { bernoulli_row1_fn(p, q, Pair.new(x, y)) })
        },
        entropy_term(bernoulli_mass(p)))
    finite_set_sum_eq_of_function_eq[Bool, Real](bernoulli_support,
        function(x: Bool) {
            finite_set_sum(bernoulli_support, function(y: Bool) { bernoulli_row1_fn(p, q, Pair.new(x, y)) })
        },
        entropy_term(bernoulli_mass(p)))
    finite_set_sum(bernoulli_support, function(x: Bool) {
            finite_set_sum(bernoulli_support, function(y: Bool) { bernoulli_row1_fn(p, q, Pair.new(x, y)) })
        }) =
        finite_set_sum(bernoulli_support, entropy_term(bernoulli_mass(p)))
    bernoulli_entropy_term_sum(p)
    finite_set_sum(bernoulli_support, entropy_term(bernoulli_mass(p))) =
        p * p.log.get_or_else(Real.0) + (Real.1 - p) * (Real.1 - p).log.get_or_else(Real.0)
    finite_set_sum(finite_set_product(bernoulli_support, bernoulli_support), bernoulli_row1_fn(p, q)) =
        p * p.log.get_or_else(Real.0) + (Real.1 - p) * (Real.1 - p).log.get_or_else(Real.0)
}

/// The sum of the second row over the product support.
theorem bernoulli_product_row2_total(p: Real, q: Real) {
    finite_set_sum(finite_set_product(bernoulli_support, bernoulli_support), bernoulli_row2_fn(p, q))
        = q * q.log.get_or_else(Real.0) + (Real.1 - q) * (Real.1 - q).log.get_or_else(Real.0)
} by {
    finite_set_sum_product_fubini_rows[Bool, Bool, Real](
        bernoulli_support, bernoulli_support, bernoulli_row2_fn(p, q))
    finite_set_sum(finite_set_product(bernoulli_support, bernoulli_support), bernoulli_row2_fn(p, q)) =
        finite_set_sum(bernoulli_support, function(x: Bool) {
            finite_set_sum(bernoulli_support, function(y: Bool) { bernoulli_row2_fn(p, q, Pair.new(x, y)) })
        })
    forall(x: Bool) {
        forall(y: Bool) {
            bernoulli_row2_fn(p, q, Pair.new(x, y)) = bernoulli_row2(p, q, x, y)
            bernoulli_row2(p, q, x, y) =
                bernoulli_mass(q, y) * (bernoulli_mass(q, y)).log.get_or_else(Real.0) * bernoulli_mass(p, x)
            entropy_term(bernoulli_mass(q), y) = bernoulli_mass(q, y) * (bernoulli_mass(q, y)).log.get_or_else(Real.0)
            mul_fn(bernoulli_mass(p, x), entropy_term(bernoulli_mass(q)), y) =
                bernoulli_mass(p, x) * entropy_term(bernoulli_mass(q), y)
            bernoulli_mass(q, y) * (bernoulli_mass(q, y)).log.get_or_else(Real.0) * bernoulli_mass(p, x) =
                bernoulli_mass(p, x) * entropy_term(bernoulli_mass(q), y)
            bernoulli_row2_fn(p, q, Pair.new(x, y)) =
                mul_fn(bernoulli_mass(p, x), entropy_term(bernoulli_mass(q)), y)
        }
        function_extensionality[Bool, Real](
            function(y: Bool) { bernoulli_row2_fn(p, q, Pair.new(x, y)) },
            mul_fn(bernoulli_mass(p, x), entropy_term(bernoulli_mass(q))))
        finite_set_sum_eq_of_function_eq[Bool, Real](bernoulli_support,
            function(y: Bool) { bernoulli_row2_fn(p, q, Pair.new(x, y)) },
            mul_fn(bernoulli_mass(p, x), entropy_term(bernoulli_mass(q))))
        finite_set_sum(bernoulli_support, function(y: Bool) { bernoulli_row2_fn(p, q, Pair.new(x, y)) }) =
            finite_set_sum(bernoulli_support, mul_fn(bernoulli_mass(p, x), entropy_term(bernoulli_mass(q))))
        finite_set_sum_scalar_mul[Bool, Real](
            bernoulli_mass(p, x), bernoulli_support, entropy_term(bernoulli_mass(q)))
        bernoulli_mass(p, x) * finite_set_sum(bernoulli_support, entropy_term(bernoulli_mass(q))) =
            finite_set_sum(bernoulli_support, mul_fn(bernoulli_mass(p, x), entropy_term(bernoulli_mass(q))))
        bernoulli_entropy_term_sum(q)
        finite_set_sum(bernoulli_support, entropy_term(bernoulli_mass(q))) =
            q * q.log.get_or_else(Real.0) + (Real.1 - q) * (Real.1 - q).log.get_or_else(Real.0)
        finite_set_sum(bernoulli_support, function(y: Bool) { bernoulli_row2_fn(p, q, Pair.new(x, y)) }) =
            bernoulli_mass(p, x) * (q * q.log.get_or_else(Real.0) + (Real.1 - q) * (Real.1 - q).log.get_or_else(Real.0))
    }
    forall(x: Bool) {
        finite_set_sum(bernoulli_support, function(y: Bool) { bernoulli_row2_fn(p, q, Pair.new(x, y)) }) =
            bernoulli_mass(p, x) * (q * q.log.get_or_else(Real.0) + (Real.1 - q) * (Real.1 - q).log.get_or_else(Real.0))
        mul_fn(q * q.log.get_or_else(Real.0) + (Real.1 - q) * (Real.1 - q).log.get_or_else(Real.0), bernoulli_mass(p), x) =
            (q * q.log.get_or_else(Real.0) + (Real.1 - q) * (Real.1 - q).log.get_or_else(Real.0)) * bernoulli_mass(p, x)
        (q * q.log.get_or_else(Real.0) + (Real.1 - q) * (Real.1 - q).log.get_or_else(Real.0)) * bernoulli_mass(p, x) =
            bernoulli_mass(p, x) * (q * q.log.get_or_else(Real.0) + (Real.1 - q) * (Real.1 - q).log.get_or_else(Real.0))
        finite_set_sum(bernoulli_support, function(y: Bool) { bernoulli_row2_fn(p, q, Pair.new(x, y)) }) =
            mul_fn(q * q.log.get_or_else(Real.0) + (Real.1 - q) * (Real.1 - q).log.get_or_else(Real.0), bernoulli_mass(p), x)
    }
    function_extensionality[Bool, Real](
        function(x: Bool) {
            finite_set_sum(bernoulli_support, function(y: Bool) { bernoulli_row2_fn(p, q, Pair.new(x, y)) })
        },
        mul_fn(q * q.log.get_or_else(Real.0) + (Real.1 - q) * (Real.1 - q).log.get_or_else(Real.0), bernoulli_mass(p)))
    finite_set_sum_eq_of_function_eq[Bool, Real](bernoulli_support,
        function(x: Bool) {
            finite_set_sum(bernoulli_support, function(y: Bool) { bernoulli_row2_fn(p, q, Pair.new(x, y)) })
        },
        mul_fn(q * q.log.get_or_else(Real.0) + (Real.1 - q) * (Real.1 - q).log.get_or_else(Real.0), bernoulli_mass(p)))
    finite_set_sum(bernoulli_support, function(x: Bool) {
            finite_set_sum(bernoulli_support, function(y: Bool) { bernoulli_row2_fn(p, q, Pair.new(x, y)) })
        }) =
        finite_set_sum(bernoulli_support,
            mul_fn(q * q.log.get_or_else(Real.0) + (Real.1 - q) * (Real.1 - q).log.get_or_else(Real.0), bernoulli_mass(p)))
    finite_set_sum_scalar_mul[Bool, Real](
        q * q.log.get_or_else(Real.0) + (Real.1 - q) * (Real.1 - q).log.get_or_else(Real.0), bernoulli_support, bernoulli_mass(p))
    (q * q.log.get_or_else(Real.0) + (Real.1 - q) * (Real.1 - q).log.get_or_else(Real.0)) *
        finite_set_sum(bernoulli_support, bernoulli_mass(p)) =
        finite_set_sum(bernoulli_support,
            mul_fn(q * q.log.get_or_else(Real.0) + (Real.1 - q) * (Real.1 - q).log.get_or_else(Real.0), bernoulli_mass(p)))
    bernoulli_support_sum(p)
    finite_set_sum(bernoulli_support, bernoulli_mass(p)) = Real.1
    (q * q.log.get_or_else(Real.0) + (Real.1 - q) * (Real.1 - q).log.get_or_else(Real.0)) * Real.1 =
        q * q.log.get_or_else(Real.0) + (Real.1 - q) * (Real.1 - q).log.get_or_else(Real.0)
    finite_set_sum(finite_set_product(bernoulli_support, bernoulli_support), bernoulli_row2_fn(p, q)) =
        q * q.log.get_or_else(Real.0) + (Real.1 - q) * (Real.1 - q).log.get_or_else(Real.0)
}

/// The entropy of the product of two independent Bernoulli distributions is the
/// sum of the binary entropies of the two factors.
theorem bernoulli_product_entropy_eq_sum(p: Real, q: Real, pmf: DiscretePMF[Pair[Bool, Bool]]) {
    Real.0 < p and p < Real.1 and Real.0 < q and q < Real.1 and
    pmf.support = finite_set_product(bernoulli_support, bernoulli_support) and
    (forall(b1: Bool, b2: Bool) {
        pmf.mass(Pair.new(b1, b2)) = bernoulli_pair_mass(p, q, Pair.new(b1, b2))
    })
    implies discrete_entropy(pmf) = binary_entropy(p) + binary_entropy(q)
} by {
    if Real.0 < p and p < Real.1 and Real.0 < q and q < Real.1 and
        pmf.support = finite_set_product(bernoulli_support, bernoulli_support) and
        (forall(b1: Bool, b2: Bool) {
            pmf.mass(Pair.new(b1, b2)) = bernoulli_pair_mass(p, q, Pair.new(b1, b2))
        }) {
        let product = finite_set_product(bernoulli_support, bernoulli_support)

        forall(pair: Pair[Bool, Bool]) {
            pair_eta(pair)
            pair = Pair.new(pair.first, pair.second)
            pmf.mass(pair) = pmf.mass(Pair.new(pair.first, pair.second))
            pmf.mass(Pair.new(pair.first, pair.second)) = bernoulli_pair_mass(p, q, Pair.new(pair.first, pair.second))
            bernoulli_pair_mass(p, q, Pair.new(pair.first, pair.second)) = bernoulli_pair_mass(p, q, pair)
            pmf.mass(pair) = bernoulli_pair_mass(p, q, pair)
        }
        function_extensionality[Pair[Bool, Bool], Real](pmf.mass, bernoulli_pair_mass(p, q))
        pmf.mass = bernoulli_pair_mass(p, q)

        forall(pair: Pair[Bool, Bool]) {
            entropy_surprise(pmf.mass, pair) = -(pmf.mass(pair) * (pmf.mass(pair)).log.get_or_else(Real.0))
            entropy_surprise(bernoulli_pair_mass(p, q), pair) =
                -(bernoulli_pair_mass(p, q, pair) * (bernoulli_pair_mass(p, q, pair)).log.get_or_else(Real.0))
            pmf.mass(pair) = bernoulli_pair_mass(p, q, pair)
            entropy_surprise(pmf.mass, pair) = entropy_surprise(bernoulli_pair_mass(p, q), pair)
        }
        function_extensionality[Pair[Bool, Bool], Real](
            entropy_surprise(pmf.mass), entropy_surprise(bernoulli_pair_mass(p, q)))
        entropy_surprise(pmf.mass) = entropy_surprise(bernoulli_pair_mass(p, q))

        discrete_entropy(pmf) = finite_set_sum(pmf.support, entropy_surprise(pmf.mass))
        discrete_entropy(pmf) = finite_set_sum(product, entropy_surprise(pmf.mass))
        finite_set_sum_eq_of_function_eq[Pair[Bool, Bool], Real](product,
            entropy_surprise(pmf.mass), entropy_surprise(bernoulli_pair_mass(p, q)))
        finite_set_sum(product, entropy_surprise(pmf.mass)) =
            finite_set_sum(product, entropy_surprise(bernoulli_pair_mass(p, q)))
        discrete_entropy(pmf) = finite_set_sum(product, entropy_surprise(bernoulli_pair_mass(p, q)))

        forall(pair: Pair[Bool, Bool]) {
            entropy_surprise(bernoulli_pair_mass(p, q), pair) = -(entropy_term(bernoulli_pair_mass(p, q), pair))
            mul_fn(-Real.1, entropy_term(bernoulli_pair_mass(p, q)), pair) =
                -Real.1 * entropy_term(bernoulli_pair_mass(p, q), pair)
            entropy_surprise(bernoulli_pair_mass(p, q), pair) =
                mul_fn(-Real.1, entropy_term(bernoulli_pair_mass(p, q)), pair)
        }
        function_extensionality[Pair[Bool, Bool], Real](
            entropy_surprise(bernoulli_pair_mass(p, q)),
            mul_fn(-Real.1, entropy_term(bernoulli_pair_mass(p, q))))
        finite_set_sum_scalar_mul[Pair[Bool, Bool], Real](-Real.1, product, entropy_term(bernoulli_pair_mass(p, q)))
        -Real.1 * finite_set_sum(product, entropy_term(bernoulli_pair_mass(p, q))) =
            finite_set_sum(product, mul_fn(-Real.1, entropy_term(bernoulli_pair_mass(p, q))))
        finite_set_sum(product, entropy_surprise(bernoulli_pair_mass(p, q))) =
            -Real.1 * finite_set_sum(product, entropy_term(bernoulli_pair_mass(p, q)))

        forall(pair: Pair[Bool, Bool]) {
            bernoulli_pair_entropy_term_split(p, q, pair)
            entropy_term(bernoulli_pair_mass(p, q), pair) =
                bernoulli_row1_fn(p, q, pair) + bernoulli_row2_fn(p, q, pair)
            add_fn(bernoulli_row1_fn(p, q), bernoulli_row2_fn(p, q), pair) =
                bernoulli_row1_fn(p, q, pair) + bernoulli_row2_fn(p, q, pair)
            entropy_term(bernoulli_pair_mass(p, q), pair) =
                add_fn(bernoulli_row1_fn(p, q), bernoulli_row2_fn(p, q), pair)
        }
        function_extensionality[Pair[Bool, Bool], Real](
            entropy_term(bernoulli_pair_mass(p, q)),
            add_fn(bernoulli_row1_fn(p, q), bernoulli_row2_fn(p, q)))
        finite_set_sum_add[Pair[Bool, Bool], Real](product, bernoulli_row1_fn(p, q), bernoulli_row2_fn(p, q))
        finite_set_sum(product, bernoulli_row1_fn(p, q)) + finite_set_sum(product, bernoulli_row2_fn(p, q)) =
            finite_set_sum(product, add_fn(bernoulli_row1_fn(p, q), bernoulli_row2_fn(p, q)))
        finite_set_sum(product, entropy_term(bernoulli_pair_mass(p, q))) =
            finite_set_sum(product, add_fn(bernoulli_row1_fn(p, q), bernoulli_row2_fn(p, q)))
        finite_set_sum(product, entropy_term(bernoulli_pair_mass(p, q))) =
            finite_set_sum(product, bernoulli_row1_fn(p, q)) + finite_set_sum(product, bernoulli_row2_fn(p, q))

        bernoulli_product_row1_total(p, q)
        finite_set_sum(product, bernoulli_row1_fn(p, q)) =
            p * p.log.get_or_else(Real.0) + (Real.1 - p) * (Real.1 - p).log.get_or_else(Real.0)
        bernoulli_product_row2_total(p, q)
        finite_set_sum(product, bernoulli_row2_fn(p, q)) =
            q * q.log.get_or_else(Real.0) + (Real.1 - q) * (Real.1 - q).log.get_or_else(Real.0)
        let a1 = p * p.log.get_or_else(Real.0) + (Real.1 - p) * (Real.1 - p).log.get_or_else(Real.0)
        let a2 = q * q.log.get_or_else(Real.0) + (Real.1 - q) * (Real.1 - q).log.get_or_else(Real.0)
        finite_set_sum(product, entropy_term(bernoulli_pair_mass(p, q))) = a1 + a2
        -Real.1 * (a1 + a2) = -(a1) + -(a2)
        discrete_entropy(pmf) = -(a1) + -(a2)
        binary_entropy(p) = -(a1)
        binary_entropy(q) = -(a2)
        binary_entropy(p) + binary_entropy(q) = -(a1) + -(a2)
        discrete_entropy(pmf) = binary_entropy(p) + binary_entropy(q)
    }
}

