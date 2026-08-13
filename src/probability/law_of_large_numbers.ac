from real import Real, lte_trans, converges_to, converges, limit, mul_seq, harmonic,
    const_converges_to, seq_pointwise_eq_converges_to, mul_seq_converges_to,
    one_over_suc_converges, limit_one_over_suc, close_imp_bounds, tail_bound,
    tail_bound_implies_is_close, abs_of_nonneg, mul_inverse, from_nat_suc_pos_real, neg_distrib
from order import lt_imp_lte, lt_of_lte_of_lt, ne_of_gt
from nat import Nat, from_nat, lt_trans, lt_suc, add_one_right, from_nat_zero, from_nat_one,
    from_nat_add
from algebra.add_semigroup import add_fn
from algebra.semigroup import mul_fn
from data.basic.functions import function_extensionality
from data.nat.nat_range_sum import range_sum, range_sum_zero, range_sum_suc, range_sum_congr
from data.nat.nat_range_sum_semiring import const_fn, semiring_count, range_sum_const,
    semiring_count_suc
from algebra.field.field import inverse_dist
from finite_set import FiniteSet
from probability.discrete_pmf import DiscretePMF, discrete_expectation, discrete_variance,
    discrete_expectation_add, discrete_expectation_scalar_mul, discrete_expectation_const,
    const_real_fn, square_fn, square_dev, pmf_event_prob, pmf_event_prob_nonneg,
    discrete_variance_formula, discrete_variance_scalar_mul, discrete_variance_add_const,
    discrete_variance_const
from probability.discrete_probability_ext import discrete_rv_independent,
    discrete_rv_independent_comm, pointwise_mul_fn, discrete_expectation_product_of_independent,
    discrete_expectation_eq_of_pointwise_eq_on_support
from probability.chebyshev import rv_abs_dev_event, chebyshev_inequality

numerals Real

/// The pointwise sum of the first `n` variables `x(0), ..., x(n - 1)` at an outcome `t`.
define range_sum_rv[T](x: Nat -> (T -> Real), n: Nat, t: T) -> Real {
    range_sum(function(i: Nat) { x(i, t) }, n)
}

/// The sample mean of the first `n + 1` variables at an outcome `t`.
define sample_mean_rv[T](x: Nat -> (T -> Real), n: Nat, t: T) -> Real {
    range_sum_rv(x, n.suc, t) / from_nat[Real](n.suc)
}

// ---------------------------------------------------------------------------
// Linearity of expectation for sums (task 2)
// ---------------------------------------------------------------------------

/// The expectation of a sum of random variables is the sum of their expectations:
/// `E(X_0 + ... + X_{n-1}) = E(X_0) + ... + E(X_{n-1})`.
theorem discrete_expectation_range_sum[T](pmf: DiscretePMF[T], x: Nat -> (T -> Real), n: Nat) {
    discrete_expectation(pmf, range_sum_rv(x, n))
        = range_sum(function(i: Nat) { discrete_expectation(pmf, x(i)) }, n)
} by {
    define p(k: Nat) -> Bool {
        discrete_expectation(pmf, range_sum_rv(x, k)) = range_sum(function(i: Nat) { discrete_expectation(pmf, x(i)) }, k)
    }

    forall(t: T) {
        range_sum_rv(x, Nat.0, t) = range_sum(function(i: Nat) { x(i, t) }, Nat.0)
        range_sum_zero(function(i: Nat) { x(i, t) })
        range_sum(function(i: Nat) { x(i, t) }, Nat.0) = Real.0
        range_sum_rv(x, Nat.0, t) = Real.0
        range_sum_rv(x, Nat.0, t) = const_real_fn[T](Real.0, t)
    }
    function_extensionality[T, Real](range_sum_rv(x, Nat.0), const_real_fn[T](Real.0))
    range_sum_rv(x, Nat.0) = const_real_fn[T](Real.0)
    discrete_expectation(pmf, range_sum_rv(x, Nat.0)) = discrete_expectation(pmf, const_real_fn[T](Real.0))
    discrete_expectation_const(pmf, Real.0)
    discrete_expectation(pmf, range_sum_rv(x, Nat.0)) = Real.0
    range_sum_zero(function(i: Nat) { discrete_expectation(pmf, x(i)) })
    range_sum(function(i: Nat) { discrete_expectation(pmf, x(i)) }, Nat.0) = Real.0
    p(Nat.0)

    forall(k: Nat) {
        if p(k) {
            forall(t: T) {
                range_sum_rv(x, k.suc, t) = range_sum(function(i: Nat) { x(i, t) }, k.suc)
                range_sum_suc(function(i: Nat) { x(i, t) }, k)
                range_sum(function(i: Nat) { x(i, t) }, k.suc) =
                    range_sum(function(i: Nat) { x(i, t) }, k) + function(i: Nat) { x(i, t) }(k)
                function(i: Nat) { x(i, t) }(k) = x(k, t)
                range_sum(function(i: Nat) { x(i, t) }, k) = range_sum_rv(x, k, t)
                range_sum_rv(x, k.suc, t) = range_sum_rv(x, k, t) + x(k, t)
                add_fn(range_sum_rv(x, k), x(k), t) = range_sum_rv(x, k, t) + x(k, t)
                range_sum_rv(x, k.suc, t) = add_fn(range_sum_rv(x, k), x(k), t)
            }
            function_extensionality[T, Real](range_sum_rv(x, k.suc), add_fn(range_sum_rv(x, k), x(k)))
            range_sum_rv(x, k.suc) = add_fn(range_sum_rv(x, k), x(k))
            discrete_expectation(pmf, range_sum_rv(x, k.suc)) =
                discrete_expectation(pmf, add_fn(range_sum_rv(x, k), x(k)))
            discrete_expectation_add(pmf, range_sum_rv(x, k), x(k))
            discrete_expectation(pmf, add_fn(range_sum_rv(x, k), x(k))) =
                discrete_expectation(pmf, range_sum_rv(x, k)) + discrete_expectation(pmf, x(k))
            p(k) = (discrete_expectation(pmf, range_sum_rv(x, k)) = range_sum(function(i: Nat) { discrete_expectation(pmf, x(i)) }, k))
            discrete_expectation(pmf, range_sum_rv(x, k)) = range_sum(function(i: Nat) { discrete_expectation(pmf, x(i)) }, k)
            discrete_expectation(pmf, range_sum_rv(x, k.suc)) =
                range_sum(function(i: Nat) { discrete_expectation(pmf, x(i)) }, k) + discrete_expectation(pmf, x(k))
            range_sum_suc(function(i: Nat) { discrete_expectation(pmf, x(i)) }, k)
            range_sum(function(i: Nat) { discrete_expectation(pmf, x(i)) }, k.suc) =
                range_sum(function(i: Nat) { discrete_expectation(pmf, x(i)) }, k) + function(i: Nat) { discrete_expectation(pmf, x(i)) }(k)
            function(i: Nat) { discrete_expectation(pmf, x(i)) }(k) = discrete_expectation(pmf, x(k))
            range_sum(function(i: Nat) { discrete_expectation(pmf, x(i)) }, k.suc) =
                range_sum(function(i: Nat) { discrete_expectation(pmf, x(i)) }, k) + discrete_expectation(pmf, x(k))
            discrete_expectation(pmf, range_sum_rv(x, k.suc)) =
                range_sum(function(i: Nat) { discrete_expectation(pmf, x(i)) }, k.suc)
            p(k.suc)
        }
    }
    forall(k: Nat) { p(k) implies p(k.suc) }
    p(Nat.0) and forall(k: Nat) { p(k) implies p(k.suc) }
    Nat.induction(p)
    p(n)
    function(x0: Nat) {
        (discrete_expectation(pmf, range_sum_rv(x, x0)) = range_sum(function(i: Nat) { discrete_expectation(pmf, x(i)) }, x0)) = p(x0)
    }(n)
    discrete_expectation(pmf, range_sum_rv(x, n)) = range_sum(function(i: Nat) { discrete_expectation(pmf, x(i)) }, n)
}

/// The semiring count of a range of naturals embeds as the natural-number real.
theorem real_semiring_count_eq_from_nat(n: Nat) {
    semiring_count[Real](n) = from_nat[Real](n)
} by {
    define p(k: Nat) -> Bool {
        semiring_count[Real](k) = from_nat[Real](k)
    }
    range_sum_zero(const_fn[Real](Real.1))
    semiring_count[Real](Nat.0) = range_sum(const_fn[Real](Real.1), Nat.0)
    semiring_count[Real](Nat.0) = Real.0
    from_nat_zero[Real]
    from_nat[Real](Nat.0) = Real.0
    p(Nat.0)
    forall(k: Nat) {
        if p(k) {
            semiring_count[Real](k) = from_nat[Real](k)
            semiring_count_suc[Real](k)
            semiring_count[Real](k.suc) = semiring_count[Real](k) + Real.1
            semiring_count[Real](k.suc) = from_nat[Real](k) + Real.1
            add_one_right(k)
            k + Nat.1 = k.suc
            from_nat_add[Real](k, Nat.1)
            from_nat[Real](k + Nat.1) = from_nat[Real](k) + from_nat[Real](Nat.1)
            from_nat_one[Real]
            from_nat[Real](Nat.1) = Real.1
            from_nat[Real](k + Nat.1) = from_nat[Real](k) + Real.1
            from_nat[Real](k.suc) = from_nat[Real](k) + Real.1
            semiring_count[Real](k.suc) = from_nat[Real](k.suc)
            p(k.suc)
        }
    }
    forall(k: Nat) { p(k) implies p(k.suc) }
    p(Nat.0) and forall(k: Nat) { p(k) implies p(k.suc) }
    Nat.induction(p)
    p(n)
    function(x0: Nat) { (semiring_count[Real](x0) = from_nat[Real](x0)) = p(x0) }(n)
    semiring_count[Real](n) = from_nat[Real](n)
}

/// A constant summand sums to the count times the constant:
/// `c + ... + c = n * c` over `n` terms.
theorem range_sum_const_real(n: Nat, c: Real) {
    range_sum(const_fn[Real](c), n) = from_nat[Real](n) * c
} by {
    range_sum_const[Real](c, n)
    range_sum(const_fn[Real](c), n) = semiring_count[Real](n) * c
    real_semiring_count_eq_from_nat(n)
    semiring_count[Real](n) = from_nat[Real](n)
    range_sum(const_fn[Real](c), n) = from_nat[Real](n) * c
}

/// The expectation of a sum of random variables of common mean `mu` is `n * mu`.
theorem discrete_expectation_range_sum_const[T](pmf: DiscretePMF[T], x: Nat -> (T -> Real), n: Nat, mu: Real) {
    (forall(i: Nat) { i < n implies discrete_expectation(pmf, x(i)) = mu }) implies
        discrete_expectation(pmf, range_sum_rv(x, n)) = from_nat[Real](n) * mu
} by {
    if forall(i: Nat) { i < n implies discrete_expectation(pmf, x(i)) = mu } {
        discrete_expectation_range_sum(pmf, x, n)
        discrete_expectation(pmf, range_sum_rv(x, n)) =
            range_sum(function(i: Nat) { discrete_expectation(pmf, x(i)) }, n)
        forall(i: Nat) {
            if i < n {
                discrete_expectation(pmf, x(i)) = mu
                function(i0: Nat) { discrete_expectation(pmf, x(i0)) }(i) = mu
                const_fn[Real](mu, i) = mu
                function(i0: Nat) { discrete_expectation(pmf, x(i0)) }(i) = const_fn[Real](mu, i)
            }
        }
        forall(i: Nat) {
            i < n implies function(j: Nat) { discrete_expectation(pmf, x(j)) }(i) = const_fn[Real](mu, i)
        }
        range_sum_congr(function(j: Nat) { discrete_expectation(pmf, x(j)) }, const_fn[Real](mu), n)
        range_sum(function(j: Nat) { discrete_expectation(pmf, x(j)) }, n) = range_sum(const_fn[Real](mu), n)
        range_sum_const_real(n, mu)
        range_sum(const_fn[Real](mu), n) = from_nat[Real](n) * mu
        discrete_expectation(pmf, range_sum_rv(x, n)) = from_nat[Real](n) * mu
    }
}

// ---------------------------------------------------------------------------
// Variance properties (task 3)
// ---------------------------------------------------------------------------

/// Variance is affine-invariant: `Var(a * X + b) = a^2 * Var(X)`.
theorem discrete_variance_affine[T](pmf: DiscretePMF[T], a: Real, x: T -> Real, b: Real) {
    discrete_variance(pmf, add_fn(mul_fn(a, x), const_real_fn[T](b))) = (a * a) * discrete_variance(pmf, x)
} by {
    discrete_variance_add_const(pmf, mul_fn(a, x), b)
    discrete_variance(pmf, add_fn(mul_fn(a, x), const_real_fn[T](b))) = discrete_variance(pmf, mul_fn(a, x))
    discrete_variance_scalar_mul(pmf, a, x)
    discrete_variance(pmf, mul_fn(a, x)) = a * a * discrete_variance(pmf, x)
    discrete_variance(pmf, add_fn(mul_fn(a, x), const_real_fn[T](b))) = a * a * discrete_variance(pmf, x)
}

/// Variance is extensional for random variables that agree pointwise on the support.
theorem discrete_variance_eq_of_pointwise_eq[T](pmf: DiscretePMF[T], x: T -> Real, y: T -> Real) {
    (forall(t: T) { pmf.support.contains(t) implies x(t) = y(t) }) implies
        discrete_variance(pmf, x) = discrete_variance(pmf, y)
} by {
    if forall(t: T) { pmf.support.contains(t) implies x(t) = y(t) } {
        discrete_expectation_eq_of_pointwise_eq_on_support(pmf, x, y)
        discrete_expectation(pmf, x) = discrete_expectation(pmf, y)
        forall(t: T) {
            if pmf.support.contains(t) {
                x(t) = y(t)
                discrete_expectation(pmf, x) = discrete_expectation(pmf, y)
                square_dev(x, discrete_expectation(pmf, x), t) = (x(t) - discrete_expectation(pmf, x)) * (x(t) - discrete_expectation(pmf, x))
                square_dev(y, discrete_expectation(pmf, y), t) = (y(t) - discrete_expectation(pmf, y)) * (y(t) - discrete_expectation(pmf, y))
                square_dev(x, discrete_expectation(pmf, x), t) = square_dev(y, discrete_expectation(pmf, y), t)
            }
        }
        forall(t: T) {
            pmf.support.contains(t) implies square_dev(x, discrete_expectation(pmf, x), t) = square_dev(y, discrete_expectation(pmf, y), t)
        }
        discrete_expectation_eq_of_pointwise_eq_on_support(pmf, square_dev(x, discrete_expectation(pmf, x)), square_dev(y, discrete_expectation(pmf, y)))
        discrete_expectation(pmf, square_dev(x, discrete_expectation(pmf, x))) = discrete_expectation(pmf, square_dev(y, discrete_expectation(pmf, y)))
        discrete_variance(pmf, x) = discrete_expectation(pmf, square_dev(x, discrete_expectation(pmf, x)))
        discrete_variance(pmf, y) = discrete_expectation(pmf, square_dev(y, discrete_expectation(pmf, y)))
        discrete_variance(pmf, x) = discrete_variance(pmf, y)
    }
}

/// The pointwise expansion of the square of a sum: `(X + Y)^2 = X^2 + 2XY + Y^2`.
define square_sum_expand[T](x: T -> Real, y: T -> Real, t: T) -> Real {
    square_fn(x, t) + mul_fn(Real.1 + Real.1, pointwise_mul_fn(x, y), t) + square_fn(y, t)
}

/// Subtracting two sums is the sum of the corresponding differences:
/// `(a + b) - (c + d) = (a - c) + (b - d)`.
lemma sum_diff_distrib(a: Real, b: Real, c: Real, d: Real) {
    (a + b) - (c + d) = (a - c) + (b - d)
} by {
    neg_distrib(c, d)
    -(c + d) = -c + -d
    (a + b) - (c + d) = a + b + -c + -d
    a - c = a + -c
    b - d = b + -d
    (a - c) + (b - d) = a + -c + b + -d
    -c + b = b + -c
    a + -c + b = a + b + -c
    a + -c + b + -d = a + b + -c + -d
}

/// A middle term cancels in a difference of sums:
/// `(a + m + b) - (c + m + d) = (a - c) + (b - d)`.
lemma cancel_middle_ring(a: Real, b: Real, c: Real, d: Real, m: Real) {
    (a + m + b) - (c + m + d) = (a - c) + (b - d)
} by {
    (a + m + b) - (c + m + d) = ((a + m) + b) - ((c + m) + d)
    sum_diff_distrib(a + m, b, c + m, d)
    ((a + m) + b) - ((c + m) + d) = ((a + m) - (c + m)) + (b - d)
    sum_diff_distrib(a, m, c, m)
    (a + m) - (c + m) = (a - c) + (m - m)
    ((a + m) - (c + m)) + (b - d) = ((a - c) + (m - m)) + (b - d)
    ((a - c) + (m - m)) + (b - d) = (a - c) + (b - d)
    (a + m + b) - (c + m + d) = (a - c) + (b - d)
}

/// The variance of the sum of two independent random variables is the sum of
/// their variances: `Var(X + Y) = Var(X) + Var(Y)`.
theorem discrete_variance_sum_independent[T](pmf: DiscretePMF[T], x: T -> Real, y: T -> Real) {
    discrete_rv_independent(pmf, x, y) implies
        discrete_variance(pmf, add_fn(x, y)) = discrete_variance(pmf, x) + discrete_variance(pmf, y)
} by {
    if discrete_rv_independent(pmf, x, y) {
        forall(t: T) {
            square_fn(add_fn(x, y), t) = add_fn(x, y, t) * add_fn(x, y, t)
            add_fn(x, y, t) = x(t) + y(t)
            square_fn(add_fn(x, y), t) = (x(t) + y(t)) * (x(t) + y(t))
            square_fn(x, t) = x(t) * x(t)
            square_fn(y, t) = y(t) * y(t)
            pointwise_mul_fn(x, y, t) = x(t) * y(t)
            mul_fn(Real.1 + Real.1, pointwise_mul_fn(x, y), t) = (Real.1 + Real.1) * pointwise_mul_fn(x, y, t)
            (Real.1 + Real.1) * (x(t) * y(t)) = x(t) * y(t) + x(t) * y(t)
            (x(t) + y(t)) * (x(t) + y(t)) = x(t) * x(t) + x(t) * y(t) + x(t) * y(t) + y(t) * y(t)
            x(t) * x(t) + x(t) * y(t) + x(t) * y(t) + y(t) * y(t) = x(t) * x(t) + (Real.1 + Real.1) * (x(t) * y(t)) + y(t) * y(t)
            square_sum_expand(x, y, t) = square_fn(x, t) + mul_fn(Real.1 + Real.1, pointwise_mul_fn(x, y), t) + square_fn(y, t)
            x(t) * x(t) + (Real.1 + Real.1) * (x(t) * y(t)) + y(t) * y(t) = square_sum_expand(x, y, t)
            (x(t) + y(t)) * (x(t) + y(t)) = square_sum_expand(x, y, t)
            add_fn(square_fn(x), add_fn(mul_fn(Real.1 + Real.1, pointwise_mul_fn(x, y)), square_fn(y)), t) =
                square_fn(x, t) + add_fn(mul_fn(Real.1 + Real.1, pointwise_mul_fn(x, y)), square_fn(y), t)
            add_fn(mul_fn(Real.1 + Real.1, pointwise_mul_fn(x, y)), square_fn(y), t) =
                mul_fn(Real.1 + Real.1, pointwise_mul_fn(x, y), t) + square_fn(y, t)
            square_sum_expand(x, y, t) = add_fn(square_fn(x), add_fn(mul_fn(Real.1 + Real.1, pointwise_mul_fn(x, y)), square_fn(y)), t)
            square_fn(add_fn(x, y), t) = add_fn(square_fn(x), add_fn(mul_fn(Real.1 + Real.1, pointwise_mul_fn(x, y)), square_fn(y)), t)
        }
        function_extensionality[T, Real](square_fn(add_fn(x, y)),
            add_fn(square_fn(x), add_fn(mul_fn(Real.1 + Real.1, pointwise_mul_fn(x, y)), square_fn(y))))
        square_fn(add_fn(x, y)) = add_fn(square_fn(x), add_fn(mul_fn(Real.1 + Real.1, pointwise_mul_fn(x, y)), square_fn(y)))
        discrete_expectation(pmf, square_fn(add_fn(x, y))) =
            discrete_expectation(pmf, add_fn(square_fn(x), add_fn(mul_fn(Real.1 + Real.1, pointwise_mul_fn(x, y)), square_fn(y))))
        discrete_expectation_add(pmf, square_fn(x), add_fn(mul_fn(Real.1 + Real.1, pointwise_mul_fn(x, y)), square_fn(y)))
        discrete_expectation(pmf, add_fn(square_fn(x), add_fn(mul_fn(Real.1 + Real.1, pointwise_mul_fn(x, y)), square_fn(y)))) =
            discrete_expectation(pmf, square_fn(x)) + discrete_expectation(pmf, add_fn(mul_fn(Real.1 + Real.1, pointwise_mul_fn(x, y)), square_fn(y)))
        discrete_expectation_add(pmf, mul_fn(Real.1 + Real.1, pointwise_mul_fn(x, y)), square_fn(y))
        discrete_expectation(pmf, add_fn(mul_fn(Real.1 + Real.1, pointwise_mul_fn(x, y)), square_fn(y))) =
            discrete_expectation(pmf, mul_fn(Real.1 + Real.1, pointwise_mul_fn(x, y))) + discrete_expectation(pmf, square_fn(y))
        discrete_expectation_scalar_mul(pmf, Real.1 + Real.1, pointwise_mul_fn(x, y))
        discrete_expectation(pmf, mul_fn(Real.1 + Real.1, pointwise_mul_fn(x, y))) = (Real.1 + Real.1) * discrete_expectation(pmf, pointwise_mul_fn(x, y))
        discrete_expectation_product_of_independent(pmf, x, y)
        discrete_expectation(pmf, pointwise_mul_fn(x, y)) = discrete_expectation(pmf, x) * discrete_expectation(pmf, y)
        discrete_expectation(pmf, square_fn(add_fn(x, y))) =
            discrete_expectation(pmf, square_fn(x)) + (Real.1 + Real.1) * (discrete_expectation(pmf, x) * discrete_expectation(pmf, y)) + discrete_expectation(pmf, square_fn(y))

        discrete_expectation_add(pmf, x, y)
        discrete_expectation(pmf, add_fn(x, y)) = discrete_expectation(pmf, x) + discrete_expectation(pmf, y)

        discrete_variance_formula(pmf, add_fn(x, y))
        discrete_variance(pmf, add_fn(x, y)) =
            discrete_expectation(pmf, square_fn(add_fn(x, y))) - discrete_expectation(pmf, add_fn(x, y)) * discrete_expectation(pmf, add_fn(x, y))
        discrete_variance_formula(pmf, x)
        discrete_variance(pmf, x) = discrete_expectation(pmf, square_fn(x)) - discrete_expectation(pmf, x) * discrete_expectation(pmf, x)
        discrete_variance_formula(pmf, y)
        discrete_variance(pmf, y) = discrete_expectation(pmf, square_fn(y)) - discrete_expectation(pmf, y) * discrete_expectation(pmf, y)

        let ex2 = discrete_expectation(pmf, square_fn(x))
        let ey2 = discrete_expectation(pmf, square_fn(y))
        let mx = discrete_expectation(pmf, x)
        let my = discrete_expectation(pmf, y)
        let ms = discrete_expectation(pmf, add_fn(x, y))
        let mxy = discrete_expectation(pmf, pointwise_mul_fn(x, y))
        ms = mx + my
        mxy = mx * my
        discrete_expectation(pmf, square_fn(add_fn(x, y))) = ex2 + (Real.1 + Real.1) * mxy + ey2
        discrete_expectation(pmf, square_fn(add_fn(x, y))) = ex2 + (Real.1 + Real.1) * (mx * my) + ey2
        discrete_variance(pmf, add_fn(x, y)) = (ex2 + (Real.1 + Real.1) * (mx * my) + ey2) - ms * ms
        (Real.1 + Real.1) * (mx * my) = mx * my + mx * my
        (mx + my) * (mx + my) = mx * mx + mx * my + mx * my + my * my
        mx * mx + mx * my + mx * my + my * my = mx * mx + (Real.1 + Real.1) * (mx * my) + my * my
        (mx + my) * (mx + my) = mx * mx + (Real.1 + Real.1) * (mx * my) + my * my
        cancel_middle_ring(ex2, ey2, mx * mx, my * my, (Real.1 + Real.1) * (mx * my))
        (ex2 + (Real.1 + Real.1) * (mx * my) + ey2) - (mx * mx + (Real.1 + Real.1) * (mx * my) + my * my) = (ex2 - mx * mx) + (ey2 - my * my)
        (ex2 + (Real.1 + Real.1) * (mx * my) + ey2) - (mx + my) * (mx + my) = (ex2 - mx * mx) + (ey2 - my * my)
        discrete_variance(pmf, add_fn(x, y)) = (ex2 - mx * mx) + (ey2 - my * my)
        (ex2 - mx * mx) = discrete_variance(pmf, x)
        (ey2 - my * my) = discrete_variance(pmf, y)
        discrete_variance(pmf, add_fn(x, y)) = discrete_variance(pmf, x) + discrete_variance(pmf, y)
    }
}

/// The variance of a range sum of sequentially independent random variables is
/// the sum of their variances.
theorem discrete_variance_range_sum_independent[T](pmf: DiscretePMF[T], x: Nat -> (T -> Real), n: Nat) {
    (forall(i: Nat) { i < n implies discrete_rv_independent(pmf, x(i), range_sum_rv(x, i)) }) implies
        discrete_variance(pmf, range_sum_rv(x, n))
            = range_sum(function(i: Nat) { discrete_variance(pmf, x(i)) }, n)
} by {
    define p(k: Nat) -> Bool {
        (forall(i: Nat) { i < k implies discrete_rv_independent(pmf, x(i), range_sum_rv(x, i)) }) implies
            discrete_variance(pmf, range_sum_rv(x, k)) = range_sum(function(i: Nat) { discrete_variance(pmf, x(i)) }, k)
    }

    forall(t: T) {
        range_sum_rv(x, Nat.0, t) = range_sum(function(i: Nat) { x(i, t) }, Nat.0)
        range_sum_zero(function(i: Nat) { x(i, t) })
        range_sum(function(i: Nat) { x(i, t) }, Nat.0) = Real.0
        range_sum_rv(x, Nat.0, t) = Real.0
        range_sum_rv(x, Nat.0, t) = const_real_fn[T](Real.0, t)
    }
    function_extensionality[T, Real](range_sum_rv(x, Nat.0), const_real_fn[T](Real.0))
    range_sum_rv(x, Nat.0) = const_real_fn[T](Real.0)
    if forall(i: Nat) { i < Nat.0 implies discrete_rv_independent(pmf, x(i), range_sum_rv(x, i)) } {
        discrete_variance_eq_of_pointwise_eq(pmf, range_sum_rv(x, Nat.0), const_real_fn[T](Real.0))
        discrete_variance(pmf, range_sum_rv(x, Nat.0)) = discrete_variance(pmf, const_real_fn[T](Real.0))
        discrete_variance_const(pmf, Real.0)
        discrete_variance(pmf, range_sum_rv(x, Nat.0)) = Real.0
        range_sum_zero(function(i: Nat) { discrete_variance(pmf, x(i)) })
        range_sum(function(i: Nat) { discrete_variance(pmf, x(i)) }, Nat.0) = Real.0
        discrete_variance(pmf, range_sum_rv(x, Nat.0)) = range_sum(function(i: Nat) { discrete_variance(pmf, x(i)) }, Nat.0)
    }
    p(Nat.0)

    forall(k: Nat) {
        if p(k) {
            if forall(i: Nat) { i < k.suc implies discrete_rv_independent(pmf, x(i), range_sum_rv(x, i)) } {
                forall(t: T) {
                    range_sum_rv(x, k.suc, t) = range_sum(function(i: Nat) { x(i, t) }, k.suc)
                    range_sum_suc(function(i: Nat) { x(i, t) }, k)
                    range_sum(function(i: Nat) { x(i, t) }, k.suc) =
                        range_sum(function(i: Nat) { x(i, t) }, k) + function(i: Nat) { x(i, t) }(k)
                    function(i: Nat) { x(i, t) }(k) = x(k, t)
                    range_sum(function(i: Nat) { x(i, t) }, k) = range_sum_rv(x, k, t)
                    range_sum_rv(x, k.suc, t) = range_sum_rv(x, k, t) + x(k, t)
                    add_fn(range_sum_rv(x, k), x(k), t) = range_sum_rv(x, k, t) + x(k, t)
                    range_sum_rv(x, k.suc, t) = add_fn(range_sum_rv(x, k), x(k), t)
                }
                function_extensionality[T, Real](range_sum_rv(x, k.suc), add_fn(range_sum_rv(x, k), x(k)))
                range_sum_rv(x, k.suc) = add_fn(range_sum_rv(x, k), x(k))
                discrete_variance_eq_of_pointwise_eq(pmf, range_sum_rv(x, k.suc), add_fn(range_sum_rv(x, k), x(k)))
                discrete_variance(pmf, range_sum_rv(x, k.suc)) = discrete_variance(pmf, add_fn(range_sum_rv(x, k), x(k)))

                forall(i: Nat) { i < k.suc implies discrete_rv_independent(pmf, x(i), range_sum_rv(x, i)) }
                lt_suc(k)
                k < k.suc
                discrete_rv_independent(pmf, x(k), range_sum_rv(x, k))
                discrete_rv_independent_comm(pmf, x(k), range_sum_rv(x, k))
                discrete_rv_independent(pmf, range_sum_rv(x, k), x(k))
                discrete_variance_sum_independent(pmf, range_sum_rv(x, k), x(k))
                discrete_variance(pmf, add_fn(range_sum_rv(x, k), x(k))) =
                    discrete_variance(pmf, range_sum_rv(x, k)) + discrete_variance(pmf, x(k))
                discrete_variance(pmf, range_sum_rv(x, k.suc)) =
                    discrete_variance(pmf, range_sum_rv(x, k)) + discrete_variance(pmf, x(k))

                forall(i: Nat) {
                    if i < k {
                        lt_trans(i, k, k.suc)
                        i < k.suc
                        discrete_rv_independent(pmf, x(i), range_sum_rv(x, i))
                    }
                }
                forall(i: Nat) { i < k implies discrete_rv_independent(pmf, x(i), range_sum_rv(x, i)) }
                p(k) = ((forall(i: Nat) { i < k implies discrete_rv_independent(pmf, x(i), range_sum_rv(x, i)) }) implies
                    discrete_variance(pmf, range_sum_rv(x, k)) = range_sum(function(i: Nat) { discrete_variance(pmf, x(i)) }, k))
                discrete_variance(pmf, range_sum_rv(x, k)) = range_sum(function(i: Nat) { discrete_variance(pmf, x(i)) }, k)
                discrete_variance(pmf, range_sum_rv(x, k.suc)) =
                    range_sum(function(i: Nat) { discrete_variance(pmf, x(i)) }, k) + discrete_variance(pmf, x(k))
                range_sum_suc(function(i: Nat) { discrete_variance(pmf, x(i)) }, k)
                range_sum(function(i: Nat) { discrete_variance(pmf, x(i)) }, k.suc) =
                    range_sum(function(i: Nat) { discrete_variance(pmf, x(i)) }, k) + function(i: Nat) { discrete_variance(pmf, x(i)) }(k)
                function(i: Nat) { discrete_variance(pmf, x(i)) }(k) = discrete_variance(pmf, x(k))
                range_sum(function(i: Nat) { discrete_variance(pmf, x(i)) }, k.suc) =
                    range_sum(function(i: Nat) { discrete_variance(pmf, x(i)) }, k) + discrete_variance(pmf, x(k))
                discrete_variance(pmf, range_sum_rv(x, k.suc)) =
                    range_sum(function(i: Nat) { discrete_variance(pmf, x(i)) }, k.suc)
                p(k.suc)
            }
            p(k.suc)
        }
    }
    forall(k: Nat) { p(k) implies p(k.suc) }
    p(Nat.0) and forall(k: Nat) { p(k) implies p(k.suc) }
    Nat.induction(p)
    p(n)
    function(x0: Nat) {
        ((forall(i: Nat) { i < x0 implies discrete_rv_independent(pmf, x(i), range_sum_rv(x, i)) }) implies
            discrete_variance(pmf, range_sum_rv(x, x0)) = range_sum(function(i: Nat) { discrete_variance(pmf, x(i)) }, x0)) = p(x0)
    }(n)
    if forall(i: Nat) { i < n implies discrete_rv_independent(pmf, x(i), range_sum_rv(x, i)) } {
        discrete_variance(pmf, range_sum_rv(x, n)) = range_sum(function(i: Nat) { discrete_variance(pmf, x(i)) }, n)
    }
    (forall(i: Nat) { i < n implies discrete_rv_independent(pmf, x(i), range_sum_rv(x, i)) }) implies discrete_variance(pmf, range_sum_rv(x, n)) = range_sum(function(i: Nat) { discrete_variance(pmf, x(i)) }, n)
}

/// The variance of a range sum of sequentially independent random variables of
/// common variance `sigma_sq` is `n * sigma_sq`.
theorem discrete_variance_range_sum_const_variance[T](pmf: DiscretePMF[T], x: Nat -> (T -> Real), n: Nat, sigma_sq: Real) {
    (forall(i: Nat) { discrete_variance(pmf, x(i)) = sigma_sq }) and
        (forall(i: Nat) { discrete_rv_independent(pmf, x(i), range_sum_rv(x, i)) })
    implies discrete_variance(pmf, range_sum_rv(x, n)) = from_nat[Real](n) * sigma_sq
} by {
    if forall(i: Nat) { discrete_variance(pmf, x(i)) = sigma_sq }
        and forall(i: Nat) { discrete_rv_independent(pmf, x(i), range_sum_rv(x, i)) } {
        forall(i: Nat) {
            if i < n {
                discrete_variance(pmf, x(i)) = sigma_sq
            }
        }
        forall(i: Nat) { i < n implies discrete_variance(pmf, x(i)) = sigma_sq }
        forall(i: Nat) {
            if i < n {
                discrete_rv_independent(pmf, x(i), range_sum_rv(x, i))
            }
        }
        forall(i: Nat) { i < n implies discrete_rv_independent(pmf, x(i), range_sum_rv(x, i)) }
        discrete_variance_range_sum_independent(pmf, x, n)
        discrete_variance(pmf, range_sum_rv(x, n)) = range_sum(function(i: Nat) { discrete_variance(pmf, x(i)) }, n)
        forall(i: Nat) {
            if i < n {
                discrete_variance(pmf, x(i)) = sigma_sq
                function(i0: Nat) { discrete_variance(pmf, x(i0)) }(i) = sigma_sq
                const_fn[Real](sigma_sq, i) = sigma_sq
                function(i0: Nat) { discrete_variance(pmf, x(i0)) }(i) = const_fn[Real](sigma_sq, i)
            }
        }
        forall(i: Nat) {
            i < n implies function(j: Nat) { discrete_variance(pmf, x(j)) }(i) = const_fn[Real](sigma_sq, i)
        }
        range_sum_congr(function(j: Nat) { discrete_variance(pmf, x(j)) }, const_fn[Real](sigma_sq), n)
        range_sum(function(j: Nat) { discrete_variance(pmf, x(j)) }, n) = range_sum(const_fn[Real](sigma_sq), n)
        range_sum_const_real(n, sigma_sq)
        range_sum(const_fn[Real](sigma_sq), n) = from_nat[Real](n) * sigma_sq
        discrete_variance(pmf, range_sum_rv(x, n)) = from_nat[Real](n) * sigma_sq
    }
}

// ---------------------------------------------------------------------------
// The weak law of large numbers (task 5)
// ---------------------------------------------------------------------------

/// The expectation of the sample mean of `n + 1` variables of common mean `mu`
/// is `mu`.
theorem sample_mean_expectation[T](pmf: DiscretePMF[T], x: Nat -> (T -> Real), n: Nat, mu: Real) {
    (forall(i: Nat) { i < n.suc implies discrete_expectation(pmf, x(i)) = mu }) implies
        discrete_expectation(pmf, sample_mean_rv(x, n)) = mu
} by {
    if forall(i: Nat) { i < n.suc implies discrete_expectation(pmf, x(i)) = mu } {
        let d = from_nat[Real](n.suc)
        let c = Real.1 / d
        forall(t: T) {
            sample_mean_rv(x, n, t) = range_sum_rv(x, n.suc, t) / d
            range_sum_rv(x, n.suc, t) / d = range_sum_rv(x, n.suc, t) * d.inverse
            mul_fn(c, range_sum_rv(x, n.suc), t) = c * range_sum_rv(x, n.suc, t)
            c = Real.1 / d
            c * range_sum_rv(x, n.suc, t) = (Real.1 / d) * range_sum_rv(x, n.suc, t)
            Real.1 / d = Real.1 * d.inverse
            Real.1 * d.inverse = d.inverse
            (Real.1 / d) * range_sum_rv(x, n.suc, t) = d.inverse * range_sum_rv(x, n.suc, t)
            d.inverse * range_sum_rv(x, n.suc, t) = range_sum_rv(x, n.suc, t) * d.inverse
            sample_mean_rv(x, n, t) = mul_fn(c, range_sum_rv(x, n.suc), t)
        }
        function_extensionality[T, Real](sample_mean_rv(x, n), mul_fn(c, range_sum_rv(x, n.suc)))
        sample_mean_rv(x, n) = mul_fn(c, range_sum_rv(x, n.suc))
        discrete_expectation(pmf, sample_mean_rv(x, n)) = discrete_expectation(pmf, mul_fn(c, range_sum_rv(x, n.suc)))
        discrete_expectation_scalar_mul(pmf, c, range_sum_rv(x, n.suc))
        discrete_expectation(pmf, mul_fn(c, range_sum_rv(x, n.suc))) = c * discrete_expectation(pmf, range_sum_rv(x, n.suc))
        discrete_expectation_range_sum_const(pmf, x, n.suc, mu)
        discrete_expectation(pmf, range_sum_rv(x, n.suc)) = from_nat[Real](n.suc) * mu
        from_nat_suc_pos_real(n)
        from_nat[Real](n.suc) > Real.0
        ne_of_gt[Real](from_nat[Real](n.suc), Real.0)
        from_nat[Real](n.suc) != Real.0
        mul_inverse(from_nat[Real](n.suc))
        from_nat[Real](n.suc) * from_nat[Real](n.suc).inverse = Real.1
        c * (from_nat[Real](n.suc) * mu) = mu
        discrete_expectation(pmf, sample_mean_rv(x, n)) = mu
    }
}

/// The variance of the sample mean of `n + 1` variables of common variance
/// `sigma_sq` is `sigma_sq / (n + 1)`.
theorem sample_mean_variance[T](pmf: DiscretePMF[T], x: Nat -> (T -> Real), n: Nat, sigma_sq: Real) {
    (forall(i: Nat) { discrete_variance(pmf, x(i)) = sigma_sq }) and
        (forall(i: Nat) { discrete_rv_independent(pmf, x(i), range_sum_rv(x, i)) })
    implies discrete_variance(pmf, sample_mean_rv(x, n)) = sigma_sq / from_nat[Real](n.suc)
} by {
    if forall(i: Nat) { discrete_variance(pmf, x(i)) = sigma_sq }
        and forall(i: Nat) { discrete_rv_independent(pmf, x(i), range_sum_rv(x, i)) } {
        let d = from_nat[Real](n.suc)
        let c = Real.1 / d
        forall(t: T) {
            sample_mean_rv(x, n, t) = range_sum_rv(x, n.suc, t) / d
            range_sum_rv(x, n.suc, t) / d = range_sum_rv(x, n.suc, t) * d.inverse
            mul_fn(c, range_sum_rv(x, n.suc), t) = c * range_sum_rv(x, n.suc, t)
            c = Real.1 / d
            c * range_sum_rv(x, n.suc, t) = (Real.1 / d) * range_sum_rv(x, n.suc, t)
            Real.1 / d = Real.1 * d.inverse
            Real.1 * d.inverse = d.inverse
            (Real.1 / d) * range_sum_rv(x, n.suc, t) = d.inverse * range_sum_rv(x, n.suc, t)
            d.inverse * range_sum_rv(x, n.suc, t) = range_sum_rv(x, n.suc, t) * d.inverse
            sample_mean_rv(x, n, t) = mul_fn(c, range_sum_rv(x, n.suc), t)
        }
        function_extensionality[T, Real](sample_mean_rv(x, n), mul_fn(c, range_sum_rv(x, n.suc)))
        sample_mean_rv(x, n) = mul_fn(c, range_sum_rv(x, n.suc))
        discrete_variance_eq_of_pointwise_eq(pmf, sample_mean_rv(x, n), mul_fn(c, range_sum_rv(x, n.suc)))
        discrete_variance(pmf, sample_mean_rv(x, n)) = discrete_variance(pmf, mul_fn(c, range_sum_rv(x, n.suc)))
        discrete_variance_scalar_mul(pmf, c, range_sum_rv(x, n.suc))
        discrete_variance(pmf, mul_fn(c, range_sum_rv(x, n.suc))) = c * c * discrete_variance(pmf, range_sum_rv(x, n.suc))
        discrete_variance_range_sum_const_variance(pmf, x, n.suc, sigma_sq)
        discrete_variance(pmf, range_sum_rv(x, n.suc)) = from_nat[Real](n.suc) * sigma_sq
        from_nat_suc_pos_real(n)
        from_nat[Real](n.suc) > Real.0
        ne_of_gt[Real](from_nat[Real](n.suc), Real.0)
        from_nat[Real](n.suc) != Real.0
        mul_inverse(from_nat[Real](n.suc))
        from_nat[Real](n.suc) * from_nat[Real](n.suc).inverse = Real.1
        from_nat[Real](n.suc).inverse * from_nat[Real](n.suc) = from_nat[Real](n.suc) * from_nat[Real](n.suc).inverse
        from_nat[Real](n.suc).inverse * from_nat[Real](n.suc) = Real.1
        c * c * (from_nat[Real](n.suc) * sigma_sq) = (from_nat[Real](n.suc).inverse * from_nat[Real](n.suc).inverse) * (from_nat[Real](n.suc) * sigma_sq)
        (from_nat[Real](n.suc).inverse * from_nat[Real](n.suc).inverse) * (from_nat[Real](n.suc) * sigma_sq) = from_nat[Real](n.suc).inverse * (from_nat[Real](n.suc).inverse * (from_nat[Real](n.suc) * sigma_sq))
        from_nat[Real](n.suc).inverse * (from_nat[Real](n.suc).inverse * (from_nat[Real](n.suc) * sigma_sq)) = from_nat[Real](n.suc).inverse * ((from_nat[Real](n.suc).inverse * from_nat[Real](n.suc)) * sigma_sq)
        from_nat[Real](n.suc).inverse * ((from_nat[Real](n.suc).inverse * from_nat[Real](n.suc)) * sigma_sq) = from_nat[Real](n.suc).inverse * (Real.1 * sigma_sq)
        from_nat[Real](n.suc).inverse * (Real.1 * sigma_sq) = from_nat[Real](n.suc).inverse * sigma_sq
        (from_nat[Real](n.suc).inverse * from_nat[Real](n.suc).inverse) * (from_nat[Real](n.suc) * sigma_sq) = from_nat[Real](n.suc).inverse * sigma_sq
        from_nat[Real](n.suc).inverse * sigma_sq = sigma_sq * from_nat[Real](n.suc).inverse
        sigma_sq * from_nat[Real](n.suc).inverse = sigma_sq / from_nat[Real](n.suc)
        c * c * (from_nat[Real](n.suc) * sigma_sq) = sigma_sq / from_nat[Real](n.suc)
        discrete_variance(pmf, mul_fn(c, range_sum_rv(x, n.suc))) = sigma_sq / from_nat[Real](n.suc)
        discrete_variance(pmf, sample_mean_rv(x, n)) = sigma_sq / from_nat[Real](n.suc)
    }
}

/// Division by a product distributes: `(a / d) / e = a / (d * e)`.
lemma div_div_eq_div_mul(a: Real, d: Real, e: Real) {
    (a / d) / e = a / (d * e)
} by {
    a / d = a * d.inverse
    (a / d) / e = (a * d.inverse) / e
    (a * d.inverse) / e = (a * d.inverse) * e.inverse
    (a * d.inverse) * e.inverse = a * (d.inverse * e.inverse)
    inverse_dist(d, e)
    (d * e).inverse = d.inverse * e.inverse
    d.inverse * e.inverse = (d * e).inverse
    a * (d.inverse * e.inverse) = a * (d * e).inverse
    a * (d * e).inverse = a / (d * e)
    (a / d) / e = a / (d * e)
}

/// A reciprocal factor times a quotient is the quotient over the product:
/// `(a / e) * (1 / d) = a / (d * e)`.
lemma mul_recip_eq_div_mul(a: Real, d: Real, e: Real) {
    (a / e) * (Real.1 / d) = a / (d * e)
} by {
    Real.1 / d = Real.1 * d.inverse
    Real.1 * d.inverse = d.inverse
    (a / e) * (Real.1 / d) = (a / e) * d.inverse
    a / e = a * e.inverse
    (a / e) * d.inverse = (a * e.inverse) * d.inverse
    (a * e.inverse) * d.inverse = a * (e.inverse * d.inverse)
    inverse_dist(d, e)
    (d * e).inverse = d.inverse * e.inverse
    e.inverse * d.inverse = d.inverse * e.inverse
    a * (e.inverse * d.inverse) = a * (d * e).inverse
    a * (d * e).inverse = a / (d * e)
    (a / e) * (Real.1 / d) = a / (d * e)
}

/// The Chebyshev tail bound for the sample mean of sequentially independent
/// variables of common mean `mu` and common variance `sigma_sq`:
/// `P(|X_0 + ... + X_n over (n + 1) - mu| >= eps) <= sigma_sq / ((n + 1) * eps^2)`.
theorem wlln_sample_mean_tail[T](pmf: DiscretePMF[T], x: Nat -> (T -> Real), mu: Real, sigma_sq: Real, eps: Real, n: Nat) {
    (forall(i: Nat) { discrete_expectation(pmf, x(i)) = mu }) and
        (forall(i: Nat) { discrete_variance(pmf, x(i)) = sigma_sq }) and
        (forall(i: Nat) { discrete_rv_independent(pmf, x(i), range_sum_rv(x, i)) }) and
        eps > Real.0
    implies
        pmf_event_prob(pmf, rv_abs_dev_event(pmf, sample_mean_rv(x, n), mu, eps))
            <= sigma_sq / (from_nat[Real](n.suc) * eps * eps)
} by {
    if forall(i: Nat) { discrete_expectation(pmf, x(i)) = mu }
        and forall(i: Nat) { discrete_variance(pmf, x(i)) = sigma_sq }
        and forall(i: Nat) { discrete_rv_independent(pmf, x(i), range_sum_rv(x, i)) }
        and eps > Real.0 {
        forall(i: Nat) {
            if i < n.suc {
                discrete_expectation(pmf, x(i)) = mu
            }
        }
        forall(i: Nat) { i < n.suc implies discrete_expectation(pmf, x(i)) = mu }
        sample_mean_expectation(pmf, x, n, mu)
        discrete_expectation(pmf, sample_mean_rv(x, n)) = mu
        chebyshev_inequality(pmf, sample_mean_rv(x, n), mu, eps)
        pmf_event_prob(pmf, rv_abs_dev_event(pmf, sample_mean_rv(x, n), mu, eps)) <= discrete_variance(pmf, sample_mean_rv(x, n)) / (eps * eps)
        forall(i: Nat) {
            if i < n.suc {
                discrete_variance(pmf, x(i)) = sigma_sq
            }
        }
        forall(i: Nat) { i < n.suc implies discrete_variance(pmf, x(i)) = sigma_sq }
        forall(i: Nat) {
            if i < n.suc {
                discrete_rv_independent(pmf, x(i), range_sum_rv(x, i))
            }
        }
        forall(i: Nat) { i < n.suc implies discrete_rv_independent(pmf, x(i), range_sum_rv(x, i)) }
        sample_mean_variance(pmf, x, n, sigma_sq)
        discrete_variance(pmf, sample_mean_rv(x, n)) = sigma_sq / from_nat[Real](n.suc)
        div_div_eq_div_mul(sigma_sq, from_nat[Real](n.suc), eps * eps)
        (sigma_sq / from_nat[Real](n.suc)) / (eps * eps) = sigma_sq / (from_nat[Real](n.suc) * (eps * eps))
        from_nat[Real](n.suc) * (eps * eps) = from_nat[Real](n.suc) * eps * eps
        (sigma_sq / from_nat[Real](n.suc)) / (eps * eps) = sigma_sq / (from_nat[Real](n.suc) * eps * eps)
        discrete_variance(pmf, sample_mean_rv(x, n)) / (eps * eps) = sigma_sq / (from_nat[Real](n.suc) * eps * eps)
        pmf_event_prob(pmf, rv_abs_dev_event(pmf, sample_mean_rv(x, n), mu, eps)) <= sigma_sq / (from_nat[Real](n.suc) * eps * eps)
    }
}

/// The WLLN tail bound as a sequence in `n`: `sigma_sq / ((n + 1) * eps^2)`.
define wlln_bound_seq(sigma_sq: Real, eps: Real, n: Nat) -> Real {
    sigma_sq / (from_nat[Real](n.suc) * eps * eps)
}

/// The WLLN tail bound tends to zero: `sigma_sq / ((n + 1) * eps^2) -> 0`.
theorem wlln_bound_seq_converges_to_zero(sigma_sq: Real, eps: Real) {
    eps > Real.0 implies converges_to(wlln_bound_seq(sigma_sq, eps), Real.0)
} by {
    if eps > Real.0 {
        one_over_suc_converges
        converges(harmonic)
        mul_seq_converges_to(sigma_sq / (eps * eps), harmonic)
        converges_to(mul_seq(sigma_sq / (eps * eps), harmonic), (sigma_sq / (eps * eps)) * limit(harmonic))
        limit_one_over_suc
        limit(harmonic) = Real.0
        (sigma_sq / (eps * eps)) * Real.0 = Real.0
        converges_to(mul_seq(sigma_sq / (eps * eps), harmonic), Real.0)
        forall(n: Nat) {
            wlln_bound_seq(sigma_sq, eps, n) = sigma_sq / (from_nat[Real](n.suc) * eps * eps)
            mul_seq(sigma_sq / (eps * eps), harmonic, n) = (sigma_sq / (eps * eps)) * harmonic(n)
            harmonic(n) = Real.1 / from_nat[Real](n.suc)
            mul_recip_eq_div_mul(sigma_sq, from_nat[Real](n.suc), eps * eps)
            (sigma_sq / (eps * eps)) * (Real.1 / from_nat[Real](n.suc)) = sigma_sq / (from_nat[Real](n.suc) * (eps * eps))
            from_nat[Real](n.suc) * (eps * eps) = from_nat[Real](n.suc) * eps * eps
            sigma_sq / (from_nat[Real](n.suc) * (eps * eps)) = sigma_sq / (from_nat[Real](n.suc) * eps * eps)
            mul_seq(sigma_sq / (eps * eps), harmonic, n) = wlln_bound_seq(sigma_sq, eps, n)
        }
        seq_pointwise_eq_converges_to(mul_seq(sigma_sq / (eps * eps), harmonic), wlln_bound_seq(sigma_sq, eps), Real.0)
        converges_to(wlln_bound_seq(sigma_sq, eps), Real.0)
    }
}

/// A nonnegative sequence bounded above by a sequence converging to zero
/// converges to zero.
theorem convergence_from_nonneg_bound(a: Nat -> Real, b: Nat -> Real) {
    (forall(n: Nat) { Real.0 <= a(n) }) and (forall(n: Nat) { a(n) <= b(n) }) and converges_to(b, Real.0)
    implies converges_to(a, Real.0)
} by {
    if forall(n: Nat) { Real.0 <= a(n) } and forall(n: Nat) { a(n) <= b(n) } and converges_to(b, Real.0) {
        forall(eps: Real) {
            if eps.is_positive {
                converges_to(b, Real.0) = forall(eps0: Real) {
                    eps0.is_positive implies exists(n: Nat) { tail_bound(b, Real.0, n, eps0) }
                }
                exists(n: Nat) { tail_bound(b, Real.0, n, eps) }
                let n: Nat satisfy { tail_bound(b, Real.0, n, eps) }
                tail_bound(a, Real.0, n, eps) = forall(i: Nat) {
                    n <= i implies a(i).is_close(Real.0, eps)
                }
                forall(i: Nat) {
                    if n <= i {
                        tail_bound_implies_is_close(b, Real.0, n, eps, i)
                        b(i).is_close(Real.0, eps)
                        close_imp_bounds(b(i), Real.0, eps)
                        b(i) < Real.0 + eps
                        Real.0 + eps = eps
                        b(i) < eps
                        a(i) <= b(i)
                        lt_of_lte_of_lt[Real](a(i), b(i), eps)
                        a(i) < eps
                        Real.0 <= a(i)
                        abs_of_nonneg(a(i))
                        a(i).abs = a(i)
                        a(i).abs < eps
                        a(i).is_close(Real.0, eps) = (a(i) - Real.0).abs < eps
                        (a(i) - Real.0).abs = a(i).abs
                        (a(i) - Real.0).abs < eps
                        a(i).is_close(Real.0, eps)
                    }
                }
                exists(n0: Nat) { tail_bound(a, Real.0, n0, eps) }
            }
        }
        forall(eps: Real) {
            eps.is_positive implies exists(n: Nat) { tail_bound(a, Real.0, n, eps) }
        }
    }
}

/// The weak law of large numbers: for sequentially independent random variables
/// of common mean `mu` and common finite variance `sigma_sq`, the probability
/// that the sample mean deviates from `mu` by at least `eps` tends to zero:
/// `P(|(X_0 + ... + X_n)/(n + 1) - mu| >= eps) -> 0` as `n -> inf`.
theorem weak_law_of_large_numbers[T](pmf: DiscretePMF[T], x: Nat -> (T -> Real), mu: Real, sigma_sq: Real, eps: Real) {
    (forall(i: Nat) { discrete_expectation(pmf, x(i)) = mu }) and
        (forall(i: Nat) { discrete_variance(pmf, x(i)) = sigma_sq }) and
        (forall(i: Nat) { discrete_rv_independent(pmf, x(i), range_sum_rv(x, i)) }) and
        eps > Real.0
    implies
        converges_to(function(n: Nat) {
            pmf_event_prob(pmf, rv_abs_dev_event(pmf, sample_mean_rv(x, n), mu, eps))
        }, Real.0)
} by {
    if forall(i: Nat) { discrete_expectation(pmf, x(i)) = mu }
        and forall(i: Nat) { discrete_variance(pmf, x(i)) = sigma_sq }
        and forall(i: Nat) { discrete_rv_independent(pmf, x(i), range_sum_rv(x, i)) }
        and eps > Real.0 {
        forall(n: Nat) {
            pmf_event_prob_nonneg(pmf, rv_abs_dev_event(pmf, sample_mean_rv(x, n), mu, eps))
            Real.0 <= pmf_event_prob(pmf, rv_abs_dev_event(pmf, sample_mean_rv(x, n), mu, eps))
        }
        forall(n: Nat) {
            Real.0 <= function(k: Nat) { pmf_event_prob(pmf, rv_abs_dev_event(pmf, sample_mean_rv(x, k), mu, eps)) }(n)
        }
        forall(n: Nat) {
            wlln_sample_mean_tail(pmf, x, mu, sigma_sq, eps, n)
            pmf_event_prob(pmf, rv_abs_dev_event(pmf, sample_mean_rv(x, n), mu, eps)) <= sigma_sq / (from_nat[Real](n.suc) * eps * eps)
            wlln_bound_seq(sigma_sq, eps, n) = sigma_sq / (from_nat[Real](n.suc) * eps * eps)
            pmf_event_prob(pmf, rv_abs_dev_event(pmf, sample_mean_rv(x, n), mu, eps)) <= wlln_bound_seq(sigma_sq, eps, n)
        }
        forall(n: Nat) {
            function(k: Nat) { pmf_event_prob(pmf, rv_abs_dev_event(pmf, sample_mean_rv(x, k), mu, eps)) }(n) <= wlln_bound_seq(sigma_sq, eps, n)
        }
        wlln_bound_seq_converges_to_zero(sigma_sq, eps)
        converges_to(wlln_bound_seq(sigma_sq, eps), Real.0)
        forall(eps0: Real) {
            if eps0.is_positive {
                converges_to(wlln_bound_seq(sigma_sq, eps), Real.0) = forall(eps1: Real) {
                    eps1.is_positive implies exists(n: Nat) { tail_bound(wlln_bound_seq(sigma_sq, eps), Real.0, n, eps1) }
                }
                exists(n: Nat) { tail_bound(wlln_bound_seq(sigma_sq, eps), Real.0, n, eps0) }
                let n: Nat satisfy { tail_bound(wlln_bound_seq(sigma_sq, eps), Real.0, n, eps0) }
                tail_bound(function(k: Nat) { pmf_event_prob(pmf, rv_abs_dev_event(pmf, sample_mean_rv(x, k), mu, eps)) }, Real.0, n, eps0) = forall(i: Nat) {
                    n <= i implies function(k: Nat) { pmf_event_prob(pmf, rv_abs_dev_event(pmf, sample_mean_rv(x, k), mu, eps)) }(i).is_close(Real.0, eps0)
                }
                forall(i: Nat) {
                    if n <= i {
                        tail_bound_implies_is_close(wlln_bound_seq(sigma_sq, eps), Real.0, n, eps0, i)
                        wlln_bound_seq(sigma_sq, eps, i).is_close(Real.0, eps0)
                        close_imp_bounds(wlln_bound_seq(sigma_sq, eps, i), Real.0, eps0)
                        wlln_bound_seq(sigma_sq, eps, i) < Real.0 + eps0
                        Real.0 + eps0 = eps0
                        wlln_bound_seq(sigma_sq, eps, i) < eps0
                        function(k: Nat) { pmf_event_prob(pmf, rv_abs_dev_event(pmf, sample_mean_rv(x, k), mu, eps)) }(i) <= wlln_bound_seq(sigma_sq, eps, i)
                        lt_of_lte_of_lt[Real](function(k: Nat) { pmf_event_prob(pmf, rv_abs_dev_event(pmf, sample_mean_rv(x, k), mu, eps)) }(i), wlln_bound_seq(sigma_sq, eps, i), eps0)
                        function(k: Nat) { pmf_event_prob(pmf, rv_abs_dev_event(pmf, sample_mean_rv(x, k), mu, eps)) }(i) < eps0
                        Real.0 <= function(k: Nat) { pmf_event_prob(pmf, rv_abs_dev_event(pmf, sample_mean_rv(x, k), mu, eps)) }(i)
                        abs_of_nonneg(function(k: Nat) { pmf_event_prob(pmf, rv_abs_dev_event(pmf, sample_mean_rv(x, k), mu, eps)) }(i))
                        function(k: Nat) { pmf_event_prob(pmf, rv_abs_dev_event(pmf, sample_mean_rv(x, k), mu, eps)) }(i).abs = function(k: Nat) { pmf_event_prob(pmf, rv_abs_dev_event(pmf, sample_mean_rv(x, k), mu, eps)) }(i)
                        function(k: Nat) { pmf_event_prob(pmf, rv_abs_dev_event(pmf, sample_mean_rv(x, k), mu, eps)) }(i).abs < eps0
                        function(k: Nat) { pmf_event_prob(pmf, rv_abs_dev_event(pmf, sample_mean_rv(x, k), mu, eps)) }(i).is_close(Real.0, eps0) = (function(k: Nat) { pmf_event_prob(pmf, rv_abs_dev_event(pmf, sample_mean_rv(x, k), mu, eps)) }(i) - Real.0).abs < eps0
                        (function(k: Nat) { pmf_event_prob(pmf, rv_abs_dev_event(pmf, sample_mean_rv(x, k), mu, eps)) }(i) - Real.0).abs = function(k: Nat) { pmf_event_prob(pmf, rv_abs_dev_event(pmf, sample_mean_rv(x, k), mu, eps)) }(i).abs
                        (function(k: Nat) { pmf_event_prob(pmf, rv_abs_dev_event(pmf, sample_mean_rv(x, k), mu, eps)) }(i) - Real.0).abs < eps0
                        function(k: Nat) { pmf_event_prob(pmf, rv_abs_dev_event(pmf, sample_mean_rv(x, k), mu, eps)) }(i).is_close(Real.0, eps0)
                    }
                }
                exists(n0: Nat) { tail_bound(function(k: Nat) { pmf_event_prob(pmf, rv_abs_dev_event(pmf, sample_mean_rv(x, k), mu, eps)) }, Real.0, n0, eps0) }
            }
        }
        forall(eps0: Real) {
            eps0.is_positive implies exists(n: Nat) { tail_bound(function(k: Nat) { pmf_event_prob(pmf, rv_abs_dev_event(pmf, sample_mean_rv(x, k), mu, eps)) }, Real.0, n, eps0) }
        }
    }
}
