/// Continuous distributions and the uniform law on [0, 1].
///
/// This file documents why the current discrete probability framework cannot
/// express continuous distributions, and what would be required to do so.
/// There are no theorems here by design; the content is an explanation for
/// future work.
///
/// ## The framework
///
/// `src/probability/` models a probability space as a finite-support
/// probability mass function: a `DiscretePMF[T]` is a finite set
/// (`pmf.support`) together with a mass function `pmf.mass : T -> Real` that
/// is nonnegative, vanishes off the support, and sums to one.  The
/// expectation of a real-valued random variable is the finite sum
///
///     E(X) = sum over pmf.support of mass(x) * X(x),
///
/// and variance, independence, and the law-of-large-numbers results in
/// `law_of_large_numbers.ac` all live on top of this finite-sum notion.
///
/// ## Why continuous distributions are not expressible
///
/// A continuous distribution — e.g. the uniform law on [0, 1] — assigns
/// probability zero to every single point and positive probability to
/// uncountable events, so it cannot be described by a finite-support mass
/// function.  Three ingredients are missing:
///
/// 1. An uncountable sample space: `DiscretePMF[T]` requires a finite
///    support, so the sample space `T` is effectively finite (or at least
///    all the mass is carried by finitely many points).
///
/// 2. A density / measure structure: probability of an event `A` would have
///    to be `integral over A of f`, for a density `f`, rather than a finite
///    sum of point masses.  The library does have Riemann integrals
///    (`src/real/integral.ac`), so the *analytic* tool exists, but there is
///    no bridge from integrals to probability: no notion of "the event
///    `X in [a, b]` as an integrable set", no expected-value-as-integral,
///    and no measure-theoretic limit theorems.
///
/// 3. No PDF-based moments: `E(X) = integral of x * f(x)` and
///    `Var(X) = E(X^2) - E(X)^2` with `E(X^2) = integral of x^2 * f(x)`
///    would need a fresh definition, plus proofs that `integral_0^1 x dx =
///    1/2` and `integral_0^1 x^2 dx = 1/3` (so the variance of the uniform
///    law is `1/3 - 1/4 = 1/12`).  Those integral facts are provable in the
///    existing Riemann machinery, but they would not *mean* anything
///    probabilistically without the missing structure.
///
/// ## What would be needed
///
/// A `ContinuousPMF` structure with a density `f : Real -> Real`, a
/// `continuous_expectation` defined through the Riemann integral, and the
/// moment computations for the uniform density `f = 1` on [0, 1]:
///
///     theorem uniform_expectation_is_one_half {
///         continuous_expectation(uniform_density, identity_fn) = Real.1 / Real.2
///     }
///
///     theorem uniform_variance_is_one_twelfth {
///         continuous_variance(uniform_density, identity_fn) = Real.1 / Real.12
///     }
///
/// until such a structure exists, the continuous case is out of scope, and
/// the discrete results in `chebyshev.ac` and `law_of_large_numbers.ac` are
/// the deliverable for the probability extension.
