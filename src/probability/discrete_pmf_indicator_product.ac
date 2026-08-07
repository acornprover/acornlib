from real import Real
from finite_set import FiniteSet, fs_intersection, finite_set_intersection_contains_eq
from data.basic.logic import and_left, and_right
from probability.discrete_pmf import DiscretePMF, finite_event_indicator,
    finite_event_indicator_one_of_contains, finite_event_indicator_zero_of_not_contains,
    discrete_expectation, discrete_expectation_indicator_eq_event_prob_support_intersection,
    pmf_event_prob, pmf_event_prob_eq_support_intersection,
    discrete_events_independent
from data.basic.functions import function_extensionality

/// The pointwise product of two finite-event indicators.
define finite_event_indicator_product[T](a: FiniteSet[T], b: FiniteSet[T], x: T) -> Real {
    finite_event_indicator(a, x) * finite_event_indicator(b, x)
}

/// The indicator of an intersection is the pointwise product of the two indicators.
theorem finite_event_indicator_intersection_eq_product[T](a: FiniteSet[T], b: FiniteSet[T], x: T) {
    finite_event_indicator(fs_intersection(a, b), x) =
        finite_event_indicator(a, x) * finite_event_indicator(b, x)
} by {
    finite_set_intersection_contains_eq(a, b, x)

    if fs_intersection(a, b).contains(x) {
        and_left(a.contains(x), b.contains(x))
        and_right(a.contains(x), b.contains(x))
        finite_event_indicator_one_of_contains(a, x)
        finite_event_indicator_one_of_contains(b, x)
        finite_event_indicator_one_of_contains(fs_intersection(a, b), x)
        finite_event_indicator(fs_intersection(a, b), x) =
            finite_event_indicator(a, x) * finite_event_indicator(b, x)
    }

    if not fs_intersection(a, b).contains(x) {
        finite_event_indicator_zero_of_not_contains(fs_intersection(a, b), x)
        if a.contains(x) {
            if b.contains(x) {
                false
            }
            finite_event_indicator_one_of_contains(a, x)
            finite_event_indicator_zero_of_not_contains(b, x)
            finite_event_indicator(b, x) = Real.0
            finite_event_indicator(a, x) * finite_event_indicator(b, x) = Real.0
        }
        if not a.contains(x) {
            finite_event_indicator_zero_of_not_contains(a, x)
            if b.contains(x) {
                finite_event_indicator_one_of_contains(b, x)
                Real.0 * finite_event_indicator(b, x) = Real.0
            }
            if not b.contains(x) {
                finite_event_indicator_zero_of_not_contains(b, x)
                function(x0: Real, x1: Real) { x0 * x1 = x1 * x0 }(Real.0, Real.0)
                Real.0 * finite_event_indicator(b, x) = Real.0
            }
            b.contains(x) or not b.contains(x)
            finite_event_indicator(a, x) * finite_event_indicator(b, x) = Real.0
        }
        a.contains(x) or not a.contains(x)
        finite_event_indicator(fs_intersection(a, b), x) =
            finite_event_indicator(a, x) * finite_event_indicator(b, x)
    }
}

/// The pointwise product function is the indicator of the event intersection.
theorem finite_event_indicator_product_eq_intersection[T](a: FiniteSet[T], b: FiniteSet[T]) {
    finite_event_indicator_product(a, b) = finite_event_indicator(fs_intersection(a, b))
} by {
    forall(x: T) {
        finite_event_indicator_intersection_eq_product(a, b, x)
        finite_event_indicator_product(a, b, x) = finite_event_indicator(fs_intersection(a, b), x)
    }
    function_extensionality[T, Real](finite_event_indicator_product(a, b), finite_event_indicator(fs_intersection(a, b)))
}

/// The expectation of the product of two finite-event indicators is the probability
/// of the intersection, restricted to the PMF support.
theorem discrete_expectation_indicator_intersection_eq_event_prob[T](
    pmf: DiscretePMF[T],
    a: FiniteSet[T],
    b: FiniteSet[T]
) {
    discrete_expectation(pmf, finite_event_indicator_product(a, b)) =
        pmf_event_prob(pmf, fs_intersection(pmf.support, fs_intersection(a, b)))
} by {
    finite_event_indicator_product_eq_intersection(a, b)
    discrete_expectation_indicator_eq_event_prob_support_intersection(pmf, fs_intersection(a, b))
}

/// The expectation of the product of two finite-event indicators is the probability
/// of the event intersection.
theorem discrete_expectation_indicator_product_eq_event_prob[T](
    pmf: DiscretePMF[T],
    a: FiniteSet[T],
    b: FiniteSet[T]
) {
    discrete_expectation(pmf, finite_event_indicator_product(a, b)) =
        pmf_event_prob(pmf, fs_intersection(a, b))
} by {
    discrete_expectation_indicator_intersection_eq_event_prob(pmf, a, b)
    discrete_expectation(pmf, finite_event_indicator_product(a, b)) =
        pmf_event_prob(pmf, fs_intersection(pmf.support, fs_intersection(a, b)))
    pmf_event_prob_eq_support_intersection(pmf, fs_intersection(a, b))
    pmf_event_prob(pmf, fs_intersection(a, b)) =
        pmf_event_prob(pmf, fs_intersection(pmf.support, fs_intersection(a, b)))
    discrete_expectation(pmf, finite_event_indicator_product(a, b)) =
        pmf_event_prob(pmf, fs_intersection(a, b))
}

/// Event independence is equivalent to factorization of the expectation of the
/// product of the corresponding finite-event indicators.
theorem discrete_events_independent_iff_indicator_product_expectation[T](
    pmf: DiscretePMF[T],
    a: FiniteSet[T],
    b: FiniteSet[T]
) {
    discrete_events_independent(pmf, a, b) =
        (discrete_expectation(pmf, finite_event_indicator_product(a, b)) =
            pmf_event_prob(pmf, a) * pmf_event_prob(pmf, b))
} by {
    discrete_expectation_indicator_product_eq_event_prob(pmf, a, b)
    discrete_expectation(pmf, finite_event_indicator_product(a, b)) =
        pmf_event_prob(pmf, fs_intersection(a, b))
    discrete_events_independent(pmf, a, b) =
        (pmf_event_prob(pmf, fs_intersection(a, b)) =
            pmf_event_prob(pmf, a) * pmf_event_prob(pmf, b))
}
