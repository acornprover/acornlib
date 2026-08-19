/// The Poisson distribution: the law of a count of rare events.
///
/// The Poisson mass function with rate `lam` assigns to each natural number `k`
/// the probability P(X = k) = e^{-lam} lam^k / k!.  The library's `DiscretePMF`
/// bundles a mass function together with a *finite* support (see
/// `discrete_pmf.ac`), so the Poisson law over the infinite outcome space
/// `Nat` is formalized here at the level of the mass function and its
/// exponential series rather than as a `DiscretePMF`.

from nat import Nat, from_nat, from_nat_zero
from rat import Rat
from list import partial, partial_scalar_mul, partial_pointwise_eq, partial_shift_suc
from real import Real, exp_term, exp_add, exp_zero, exp_term_zero_index, exp_term_one_index, exp_term_partial_converges, exp_term_mul_recurrence, exp_pos, exp_term_pos, converges, converges_to, limit, converges_imp_converges_to, converges_to_imp_converges, converges_to_unique, convergent_converges_to_limit, mul_seq, mul_seq_converges_to, tail, tail_imp_converges_to, seq_pointwise_eq_converges_to, from_nat_is_from_rat, mul_pos_pos, gt_zero_imp_pos
from algebra.semigroup import mul_fn
from data.basic.functions import function_extensionality, compose

/// The Poisson probability mass with rate `lam` at `k`:
/// P(X = k) = e^{-lam} lam^k / k! for every natural number `k`.
define poisson_mass(lam: Real, k: Nat) -> Real {
    (-lam).exp * exp_term(lam, k)
}

/// The k-th summand of the Poisson expectation: k e^{-lam} lam^k / k!.
define poisson_mean_term(lam: Real, k: Nat) -> Real {
    from_nat[Real](k) * poisson_mass(lam, k)
}

/// The partial sums of the Poisson series converge, to
/// e^{-lam} times the limit of the exponential series (which is 1):
/// `converges_to(partial(poisson_mass(lam)), (-lam).exp * limit(partial(exp_term(lam))))`.
theorem poisson_series_converges_to_one(lam: Real) {
    converges_to(partial(poisson_mass(lam)), (-lam).exp * limit(partial(exp_term(lam))))
} by {
    // The exponential series converges.
    exp_term_partial_converges(lam)
    converges(partial(exp_term(lam)))

    // Scaling the exponential series by e^{-lam} preserves convergence to
    // e^{-lam} times the limit of the exponential series.
    mul_seq_converges_to((-lam).exp, partial(exp_term(lam)))
    converges_to(mul_seq((-lam).exp, partial(exp_term(lam))), (-lam).exp * limit(partial(exp_term(lam))))

    // The Poisson series is the pointwise-scaled exponential series:
    // e^{-lam} lam^k / k! = e^{-lam} * exp_term(lam, k) for every k.
    forall(k: Nat) {
        poisson_mass(lam, k) = (-lam).exp * exp_term(lam, k)
        mul_fn((-lam).exp, exp_term(lam), k) = (-lam).exp * exp_term(lam, k)
        poisson_mass(lam, k) = mul_fn((-lam).exp, exp_term(lam), k)
    }
    forall(k: Nat) { poisson_mass(lam, k) = mul_fn((-lam).exp, exp_term(lam), k) }

    // The partial sums therefore agree with the scaled partial sums of the
    // exponential series.
    forall(n: Nat) {
        forall(k: Nat) {
            if k < n {
                poisson_mass(lam, k) = mul_fn((-lam).exp, exp_term(lam), k)
            }
        }
        forall(k: Nat) { k < n implies poisson_mass(lam, k) = mul_fn((-lam).exp, exp_term(lam), k) }
        partial_pointwise_eq(poisson_mass(lam), mul_fn((-lam).exp, exp_term(lam)), n)
        partial(poisson_mass(lam), n) = partial(mul_fn((-lam).exp, exp_term(lam)), n)
        partial_scalar_mul((-lam).exp, exp_term(lam), n)
        (-lam).exp * partial(exp_term(lam), n) = partial(mul_fn((-lam).exp, exp_term(lam)), n)
        partial(poisson_mass(lam), n) = (-lam).exp * partial(exp_term(lam), n)
        mul_seq((-lam).exp, partial(exp_term(lam)), n) = (-lam).exp * partial(exp_term(lam), n)
        partial(poisson_mass(lam), n) = mul_seq((-lam).exp, partial(exp_term(lam)), n)
    }
    forall(n: Nat) {
        partial(poisson_mass(lam), n) = mul_seq((-lam).exp, partial(exp_term(lam)), n)
    }
    seq_pointwise_eq_converges_to(mul_seq((-lam).exp, partial(exp_term(lam))), partial(poisson_mass(lam)), (-lam).exp * limit(partial(exp_term(lam))))
    converges_to(partial(poisson_mass(lam)), (-lam).exp * limit(partial(exp_term(lam))))
}

/// The partial sums of the Poisson series converge.
theorem poisson_series_converges(lam: Real) {
    converges(partial(poisson_mass(lam)))
} by {
    poisson_series_converges_to_one(lam)
    converges_to(partial(poisson_mass(lam)), (-lam).exp * limit(partial(exp_term(lam))))
    converges_to_imp_converges(partial(poisson_mass(lam)), (-lam).exp * limit(partial(exp_term(lam))))
    converges(partial(poisson_mass(lam)))
}

/// The Poisson probabilities sum to one: Σ_{k>=0} e^{-lam} lam^k / k! = 1.
/// The limit of the partial sums of the Poisson series equals
/// e^{-lam} e^lam = e^0 = 1.
theorem poisson_series_sums_to_one(lam: Real) {
    limit(partial(poisson_mass(lam))) = Real.1
} by {
    // The partial sums converge, so their limit is characterized by
    // converges_to.
    poisson_series_converges(lam)
    converges(partial(poisson_mass(lam)))
    convergent_converges_to_limit(partial(poisson_mass(lam)))
    converges_to(partial(poisson_mass(lam)), limit(partial(poisson_mass(lam))))
    poisson_series_converges_to_one(lam)
    converges_to(partial(poisson_mass(lam)), (-lam).exp * limit(partial(exp_term(lam))))
    converges_to_unique(partial(poisson_mass(lam)), limit(partial(poisson_mass(lam))), (-lam).exp * limit(partial(exp_term(lam))))
    limit(partial(poisson_mass(lam))) = (-lam).exp * limit(partial(exp_term(lam)))

    // The limit of the exponential series is lam.exp.
    lam.exp = limit(partial(exp_term(lam)))
    limit(partial(poisson_mass(lam))) = (-lam).exp * lam.exp

    // e^{-lam} e^lam = e^0 = 1 by the addition law.
    exp_add(-lam, lam)
    (-lam + lam).exp = (-lam).exp * lam.exp
    -lam + lam = Real.0
    (Real.0).exp = (-lam).exp * lam.exp
    exp_zero
    Real.1 = (-lam).exp * lam.exp
    (-lam).exp * lam.exp = Real.1
    limit(partial(poisson_mass(lam))) = Real.1
}

/// The probability that a Poisson(lam) variable equals zero is e^{-lam}.
theorem poisson_mass_zero(lam: Real) {
    poisson_mass(lam, Nat.0) = (-lam).exp
} by {
    poisson_mass(lam, Nat.0) = (-lam).exp * exp_term(lam, Nat.0)
    exp_term_zero_index(lam)
    exp_term(lam, Nat.0) = Real.1
    (-lam).exp * Real.1 = (-lam).exp
}

/// The probability that a Poisson(lam) variable equals one is lam e^{-lam}.
theorem poisson_mass_one(lam: Real) {
    poisson_mass(lam, Nat.1) = lam * (-lam).exp
} by {
    poisson_mass(lam, Nat.1) = (-lam).exp * exp_term(lam, Nat.1)
    exp_term_one_index(lam)
    exp_term(lam, Nat.1) = lam
    (-lam).exp * lam = lam * (-lam).exp
}

/// For a positive rate, every Poisson mass is positive: the exponential
/// factor e^{-lam} and every series term lam^k / k! are positive.
theorem poisson_mass_pos(lam: Real, k: Nat) {
    lam > Real.0 implies Real.0 < poisson_mass(lam, k)
} by {
    if lam > Real.0 {
        exp_pos(-lam)
        (-lam).exp > Real.0
        gt_zero_imp_pos((-lam).exp)
        (-lam).exp.is_positive
        exp_term_pos(lam, k)
        lam > Real.0 implies exp_term(lam, k) > Real.0
        exp_term(lam, k) > Real.0
        gt_zero_imp_pos(exp_term(lam, k))
        exp_term(lam, k).is_positive
        mul_pos_pos((-lam).exp, exp_term(lam, k))
        ((-lam).exp * exp_term(lam, k)).is_positive
        Real.0 < (-lam).exp * exp_term(lam, k)
        poisson_mass(lam, k) = (-lam).exp * exp_term(lam, k)
        Real.0 < poisson_mass(lam, k)
    }
}

/// The shifted Poisson mean term satisfies
/// (k+1) e^{-lam} lam^{k+1} / (k+1)! = lam e^{-lam} lam^k / k!.
/// This is the index-shift that turns the expectation series into lam times
/// the Poisson series.
lemma poisson_mean_term_shift(lam: Real, k: Nat) {
    poisson_mean_term(lam, k.suc) = lam * poisson_mass(lam, k)
} by {
    poisson_mean_term(lam, k.suc) = from_nat[Real](k.suc) * poisson_mass(lam, k.suc)
    poisson_mass(lam, k.suc) = (-lam).exp * exp_term(lam, k.suc)

    // (k+1) * exp_term(lam, k+1) = lam * exp_term(lam, k) by the recurrence.
    from_nat_is_from_rat(k.suc)
    from_nat[Real](k.suc) = Real.from_rat(Rat.from_nat(k.suc))
    exp_term_mul_recurrence(lam, k)
    exp_term(lam, k.suc) * Real.from_rat(Rat.from_nat(k.suc)) = lam * exp_term(lam, k)
    Real.from_rat(Rat.from_nat(k.suc)) * exp_term(lam, k.suc) = lam * exp_term(lam, k)
    from_nat[Real](k.suc) * exp_term(lam, k.suc) = lam * exp_term(lam, k)

    from_nat[Real](k.suc) * ((-lam).exp * exp_term(lam, k.suc)) =
        (-lam).exp * (from_nat[Real](k.suc) * exp_term(lam, k.suc))
    poisson_mean_term(lam, k.suc) =
        (-lam).exp * (from_nat[Real](k.suc) * exp_term(lam, k.suc))
    poisson_mean_term(lam, k.suc) = (-lam).exp * (lam * exp_term(lam, k))
    (-lam).exp * (lam * exp_term(lam, k)) = lam * ((-lam).exp * exp_term(lam, k))
    poisson_mean_term(lam, k.suc) = lam * ((-lam).exp * exp_term(lam, k))
    poisson_mass(lam, k) = (-lam).exp * exp_term(lam, k)
    poisson_mean_term(lam, k.suc) = lam * poisson_mass(lam, k)
}

/// The mean of a Poisson(lam) random variable is lam:
/// E[X] = Σ_{k>=0} k e^{-lam} lam^k / k! = lam.
/// The series identity is the index shift of the previous lemma:
/// Σ k e^{-lam} lam^k / k! = e^{-lam} lam Σ lam^{k-1} / (k-1)!
///                          = lam e^{-lam} e^{lam} = lam.
theorem poisson_mean(lam: Real) {
    limit(partial(poisson_mean_term(lam))) = lam
} by {
    // The (n+1)-st partial sum of the mean series is lam times the n-th
    // partial sum of the Poisson series.
    forall(n: Nat) {
        partial_shift_suc(poisson_mean_term(lam), n)
        poisson_mean_term(lam, Nat.0) + partial(compose(poisson_mean_term(lam), Nat.suc), n) =
            partial(poisson_mean_term(lam), n.suc)

        // The k = 0 term of the mean series vanishes.
        poisson_mean_term(lam, Nat.0) = from_nat[Real](Nat.0) * poisson_mass(lam, Nat.0)
        from_nat_zero[Real]
        from_nat[Real](Nat.0) = Real.0
        Real.0 * poisson_mass(lam, Nat.0) = Real.0
        poisson_mean_term(lam, Nat.0) = Real.0

        // The shifted terms are exactly the lam-scaled Poisson masses.
        forall(k: Nat) {
            poisson_mean_term_shift(lam, k)
            poisson_mean_term(lam, k.suc) = lam * poisson_mass(lam, k)
            compose(poisson_mean_term(lam), Nat.suc, k) = poisson_mean_term(lam, Nat.suc(k))
            Nat.suc(k) = k.suc
            compose(poisson_mean_term(lam), Nat.suc, k) = poisson_mean_term(lam, k.suc)
            mul_fn(lam, poisson_mass(lam), k) = lam * poisson_mass(lam, k)
            compose(poisson_mean_term(lam), Nat.suc, k) = mul_fn(lam, poisson_mass(lam), k)
        }
        forall(k: Nat) {
            compose(poisson_mean_term(lam), Nat.suc, k) = mul_fn(lam, poisson_mass(lam), k)
        }
        function_extensionality(compose(poisson_mean_term(lam), Nat.suc), mul_fn(lam, poisson_mass(lam)))
        compose(poisson_mean_term(lam), Nat.suc) = mul_fn(lam, poisson_mass(lam))
        partial(compose(poisson_mean_term(lam), Nat.suc), n) =
            partial(mul_fn(lam, poisson_mass(lam)), n)
        partial_scalar_mul(lam, poisson_mass(lam), n)
        lam * partial(poisson_mass(lam), n) = partial(mul_fn(lam, poisson_mass(lam)), n)
        partial(compose(poisson_mean_term(lam), Nat.suc), n) = lam * partial(poisson_mass(lam), n)

        Real.0 + lam * partial(poisson_mass(lam), n) = lam * partial(poisson_mass(lam), n)
        partial(poisson_mean_term(lam), n.suc) = lam * partial(poisson_mass(lam), n)

        mul_seq(lam, partial(poisson_mass(lam)), n) = lam * partial(poisson_mass(lam), n)
        partial(poisson_mean_term(lam), n.suc) = mul_seq(lam, partial(poisson_mass(lam)), n)
        compose(partial(poisson_mean_term(lam)), Nat.suc, n) =
            partial(poisson_mean_term(lam), Nat.suc(n))
        Nat.suc(n) = n.suc
        compose(partial(poisson_mean_term(lam)), Nat.suc, n) = partial(poisson_mean_term(lam), n.suc)
        compose(partial(poisson_mean_term(lam)), Nat.suc, n) = mul_seq(lam, partial(poisson_mass(lam)), n)
    }
    forall(n: Nat) {
        compose(partial(poisson_mean_term(lam)), Nat.suc, n) = mul_seq(lam, partial(poisson_mass(lam)), n)
    }

    // The lam-scaled Poisson partial sums converge to lam * 1 = lam.
    poisson_series_converges(lam)
    converges(partial(poisson_mass(lam)))
    mul_seq_converges_to(lam, partial(poisson_mass(lam)))
    converges_to(mul_seq(lam, partial(poisson_mass(lam))), lam * limit(partial(poisson_mass(lam))))
    seq_pointwise_eq_converges_to(mul_seq(lam, partial(poisson_mass(lam))),
        compose(partial(poisson_mean_term(lam)), Nat.suc), lam * limit(partial(poisson_mass(lam))))
    converges_to(compose(partial(poisson_mean_term(lam)), Nat.suc), lam * limit(partial(poisson_mass(lam))))

    // The shifted partial sums are the tail of the mean series.
    forall(i: Nat) {
        tail(partial(poisson_mean_term(lam)), Nat.1, i) = partial(poisson_mean_term(lam), Nat.1 + i)
        Nat.1 + i = i.suc
        compose(partial(poisson_mean_term(lam)), Nat.suc, i) = partial(poisson_mean_term(lam), i.suc)
        tail(partial(poisson_mean_term(lam)), Nat.1, i) =
            compose(partial(poisson_mean_term(lam)), Nat.suc, i)
    }
    forall(i: Nat) {
        tail(partial(poisson_mean_term(lam)), Nat.1, i) =
            compose(partial(poisson_mean_term(lam)), Nat.suc, i)
    }
    seq_pointwise_eq_converges_to(compose(partial(poisson_mean_term(lam)), Nat.suc),
        tail(partial(poisson_mean_term(lam)), Nat.1), lam * limit(partial(poisson_mass(lam))))
    converges_to(tail(partial(poisson_mean_term(lam)), Nat.1), lam * limit(partial(poisson_mass(lam))))

    // A convergent tail makes the whole sequence converge, to the tail's limit.
    converges_to_imp_converges(tail(partial(poisson_mean_term(lam)), Nat.1), lam * limit(partial(poisson_mass(lam))))
    converges(tail(partial(poisson_mean_term(lam)), Nat.1))
    tail_imp_converges_to(partial(poisson_mean_term(lam)), Nat.1)
    converges_to(partial(poisson_mean_term(lam)), limit(tail(partial(poisson_mean_term(lam)), Nat.1)))
    convergent_converges_to_limit(tail(partial(poisson_mean_term(lam)), Nat.1))
    converges_to(tail(partial(poisson_mean_term(lam)), Nat.1), limit(tail(partial(poisson_mean_term(lam)), Nat.1)))
    converges_to_unique(tail(partial(poisson_mean_term(lam)), Nat.1),
        lam * limit(partial(poisson_mass(lam))), limit(tail(partial(poisson_mean_term(lam)), Nat.1)))
    lam * limit(partial(poisson_mass(lam))) = limit(tail(partial(poisson_mean_term(lam)), Nat.1))
    converges_to(partial(poisson_mean_term(lam)), lam * limit(partial(poisson_mass(lam))))

    // The mean series converges, so its limit is characterized by converges_to.
    converges_to_imp_converges(partial(poisson_mean_term(lam)), lam * limit(partial(poisson_mass(lam))))
    converges(partial(poisson_mean_term(lam)))
    convergent_converges_to_limit(partial(poisson_mean_term(lam)))
    converges_to(partial(poisson_mean_term(lam)), limit(partial(poisson_mean_term(lam))))
    converges_to_unique(partial(poisson_mean_term(lam)), limit(partial(poisson_mean_term(lam))),
        lam * limit(partial(poisson_mass(lam))))
    limit(partial(poisson_mean_term(lam))) = lam * limit(partial(poisson_mass(lam)))

    // The Poisson series sums to one.
    poisson_series_sums_to_one(lam)
    limit(partial(poisson_mass(lam))) = Real.1
    limit(partial(poisson_mean_term(lam))) = lam * Real.1
    lam * Real.1 = lam
    limit(partial(poisson_mean_term(lam))) = lam
}

// The variance of a Poisson(lam) random variable is also lam.
// Var(X) = E[X^2] - E[X]^2 = lam, and E[X^2] = lam^2 + lam, so
// Var(X) = lam: the mean equals the variance, the signature of the
// Poisson law.  The proof would proceed along the same series route as the
// mean: the second-moment series Σ k^2 e^{-lam} lam^k / k! splits, via the
// identity k^2 = k(k-1) + k, into lam^2 Σ e^{-lam} lam^{k-2} / (k-2)!
// (shifted twice) plus the mean series, giving E[X^2] = lam^2 + lam.

// The Poisson limit of the binomial (the law of rare events): as n -> infinity,
// Binomial(n, lam/n) converges in distribution to Poisson(lam).  Formally:
// for every fixed k, the binomial mass
//   P(X_n = k) = C(n, k) (lam/n)^k (1 - lam/n)^{n-k}
// converges to e^{-lam} lam^k / k!.  The proof needs a limit formulation for
// binomial masses and the exponential limit (1 - lam/n)^n -> e^{-lam}, which
// the library does not yet have in the form needed here; it is left as a
// statement.

