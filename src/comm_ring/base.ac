from algebra.comm_monoid import CommMonoid
from algebra.ring.ring import Ring

/// A commutative ring is a ring where multiplication is also commutative.
typeclass CommRing extends Ring, CommMonoid