/// The geometric sum identity: `a^n - b^n` factorizes as
/// `(a - b) * (a^(n-1) + a^(n-2) b + ... + b^(n-1))` in any commutative ring.
///
/// The summand `a^k * b^(n - k)` is indexed by the power of `a`; the proof is
/// by induction on `n`, peeling the leading term `a^n` with the split-last
/// partial-sum lemma and applying the induction hypothesis to the remaining
/// `b * (a^(n-1) + ... + b^(n-1))`.

from comm_ring.base import CommRing
from nat import Nat, alt_induction, lt_suc, suc_sub_one, sub_zero, sub_self,
    add_imp_sub, add_sub, zero_or_suc
from order import lt_imp_lte
from algebra.add_comm_monoid import AddCommMonoid
from algebra.semigroup import mul_fn
from data.basic.functions import compose
from list import partial, partial_zero, partial_split_last, partial_scalar_mul, partial_pointwise_eq

numerals Nat

/// A term of the geometric sum: `a^k * b^(n - k)`.
define geom_sum_term[R: CommRing](a: R, b: R, n: Nat, k: Nat) -> R {
    a.pow(k) * b.pow(n - k)
}

/// A successor difference is the successor of the difference.
lemma geom_nat_suc_sub(a: Nat, b: Nat) {
    b <= a implies a.suc - b = (a - b).suc
} by {
    if b <= a {
        add_sub(a, b)
        a - b + b = a
        (a - b + b).suc = a.suc
        (a - b + b).suc = (a - b).suc + b
        (a - b).suc + b = a.suc
        add_imp_sub((a - b).suc, b, a.suc)
        a.suc - b = (a - b).suc
    }
}

/// The successor row of the geometric sum peels off the leading `a^(n+1)`
/// term.
theorem geom_sum_term_step[R: CommRing](a: R, b: R, n: Nat) {
    partial(geom_sum_term[R](a, b, n.suc), n.suc.suc) =
        a.pow(n.suc) + b * partial(geom_sum_term[R](a, b, n), n.suc)
} by {
    partial_split_last(geom_sum_term[R](a, b, n.suc), n.suc)
    partial(geom_sum_term[R](a, b, n.suc), n.suc.suc) =
        partial(geom_sum_term[R](a, b, n.suc), n.suc) + geom_sum_term[R](a, b, n.suc, n.suc)
    geom_sum_term[R](a, b, n.suc, n.suc) = a.pow(n.suc) * b.pow(n.suc - n.suc)
    sub_self(n.suc)
    n.suc - n.suc = Nat.0
    b.pow(n.suc - n.suc) = R.1
    geom_sum_term[R](a, b, n.suc, n.suc) = a.pow(n.suc) * R.1
    a.pow(n.suc) * R.1 = a.pow(n.suc)
    geom_sum_term[R](a, b, n.suc, n.suc) = a.pow(n.suc)
    partial(geom_sum_term[R](a, b, n.suc), n.suc.suc) =
        partial(geom_sum_term[R](a, b, n.suc), n.suc) + a.pow(n.suc)
    forall(j: Nat) {
        if j < n.suc {
            geom_sum_term[R](a, b, n.suc, j) = a.pow(j) * b.pow(n.suc - j)
            lt_imp_lte(j, n)
            j <= n
            geom_nat_suc_sub(n, j)
            n.suc - j = (n - j).suc
            b.pow(n.suc - j) = b.pow((n - j).suc)
            b.pow((n - j).suc) = b * b.pow(n - j)
            geom_sum_term[R](a, b, n.suc, j) = a.pow(j) * (b * b.pow(n - j))
            a.pow(j) * (b * b.pow(n - j)) = b * (a.pow(j) * b.pow(n - j))
            geom_sum_term[R](a, b, n, j) = a.pow(j) * b.pow(n - j)
            geom_sum_term[R](a, b, n.suc, j) = b * geom_sum_term[R](a, b, n, j)
            mul_fn(b, geom_sum_term[R](a, b, n), j) = b * geom_sum_term[R](a, b, n, j)
            geom_sum_term[R](a, b, n.suc, j) = mul_fn(b, geom_sum_term[R](a, b, n), j)
        }
    }
    partial_pointwise_eq(geom_sum_term[R](a, b, n.suc), mul_fn(b, geom_sum_term[R](a, b, n)), n.suc)
    partial(geom_sum_term[R](a, b, n.suc), n.suc) =
        partial(mul_fn(b, geom_sum_term[R](a, b, n)), n.suc)
    partial_scalar_mul(b, geom_sum_term[R](a, b, n), n.suc)
    b * partial(geom_sum_term[R](a, b, n), n.suc) =
        partial(mul_fn(b, geom_sum_term[R](a, b, n)), n.suc)
    partial(geom_sum_term[R](a, b, n.suc), n.suc) =
        b * partial(geom_sum_term[R](a, b, n), n.suc)
    partial(geom_sum_term[R](a, b, n.suc), n.suc.suc) =
        a.pow(n.suc) + b * partial(geom_sum_term[R](a, b, n), n.suc)
}

/// The difference of powers identity: `(a - b) * (a^n + a^(n-1) b + ... +
/// b^n) = a^(n+1) - b^(n+1)`.
theorem geom_sum[R: CommRing](a: R, b: R, n: Nat) {
    (a - b) * partial(geom_sum_term[R](a, b, n), n.suc) = a.pow(n.suc) - b.pow(n.suc)
} by {
    define statement(k: Nat) -> Bool {
        (a - b) * partial(geom_sum_term[R](a, b, k), k.suc) = a.pow(k.suc) - b.pow(k.suc)
    }

    partial(geom_sum_term[R](a, b, Nat.0), Nat.0.suc) = geom_sum_term[R](a, b, Nat.0, Nat.0)
    geom_sum_term[R](a, b, Nat.0, Nat.0) = a.pow(Nat.0) * b.pow(Nat.0 - Nat.0)
    sub_self(Nat.0)
    Nat.0 - Nat.0 = Nat.0
    a.pow(Nat.0) = R.1
    b.pow(Nat.0) = R.1
    geom_sum_term[R](a, b, Nat.0, Nat.0) = R.1 * R.1
    R.1 * R.1 = R.1
    partial(geom_sum_term[R](a, b, Nat.0), Nat.0.suc) = R.1
    (a - b) * partial(geom_sum_term[R](a, b, Nat.0), Nat.0.suc) = (a - b) * R.1
    (a - b) * R.1 = a - b
    a.pow(Nat.0.suc) = a * a.pow(Nat.0)
    a.pow(Nat.0) = R.1
    a.pow(Nat.0.suc) = a * R.1
    a * R.1 = a
    a.pow(Nat.0.suc) = a
    b.pow(Nat.0.suc) = b * b.pow(Nat.0)
    b.pow(Nat.0) = R.1
    b.pow(Nat.0.suc) = b * R.1
    b * R.1 = b
    b.pow(Nat.0.suc) = b
    a.pow(Nat.0.suc) - b.pow(Nat.0.suc) = a - b
    (a - b) * partial(geom_sum_term[R](a, b, Nat.0), Nat.0.suc) =
        a.pow(Nat.0.suc) - b.pow(Nat.0.suc)
    statement(Nat.0)

    forall(k: Nat) {
        if statement(k) {
            geom_sum_term_step(a, b, k)
            partial(geom_sum_term[R](a, b, k.suc), k.suc.suc) =
                a.pow(k.suc) + b * partial(geom_sum_term[R](a, b, k), k.suc)
            (a - b) * partial(geom_sum_term[R](a, b, k.suc), k.suc.suc) =
                (a - b) * (a.pow(k.suc) + b * partial(geom_sum_term[R](a, b, k), k.suc))
            (a - b) * (a.pow(k.suc) + b * partial(geom_sum_term[R](a, b, k), k.suc)) =
                (a - b) * a.pow(k.suc) + (a - b) * (b * partial(geom_sum_term[R](a, b, k), k.suc))
            (a - b) * (b * partial(geom_sum_term[R](a, b, k), k.suc)) =
                b * ((a - b) * partial(geom_sum_term[R](a, b, k), k.suc))
            (a - b) * partial(geom_sum_term[R](a, b, k.suc), k.suc.suc) =
                (a - b) * a.pow(k.suc) + b * ((a - b) * partial(geom_sum_term[R](a, b, k), k.suc))
            statement(k) = ((a - b) * partial(geom_sum_term[R](a, b, k), k.suc) =
                a.pow(k.suc) - b.pow(k.suc))
            (a - b) * partial(geom_sum_term[R](a, b, k), k.suc) = a.pow(k.suc) - b.pow(k.suc)
            (a - b) * partial(geom_sum_term[R](a, b, k.suc), k.suc.suc) =
                (a - b) * a.pow(k.suc) + b * (a.pow(k.suc) - b.pow(k.suc))
            (a - b) * a.pow(k.suc) = a * a.pow(k.suc) - b * a.pow(k.suc)
            a * a.pow(k.suc) = a.pow(k.suc.suc)
            (a - b) * a.pow(k.suc) = a.pow(k.suc.suc) - b * a.pow(k.suc)
            b * (a.pow(k.suc) - b.pow(k.suc)) = b * a.pow(k.suc) - b * b.pow(k.suc)
            b * b.pow(k.suc) = b.pow(k.suc.suc)
            b * (a.pow(k.suc) - b.pow(k.suc)) = b * a.pow(k.suc) - b.pow(k.suc.suc)
            (a - b) * a.pow(k.suc) + b * (a.pow(k.suc) - b.pow(k.suc)) =
                (a.pow(k.suc.suc) - b * a.pow(k.suc)) + (b * a.pow(k.suc) - b.pow(k.suc.suc))
            (a.pow(k.suc.suc) - b * a.pow(k.suc)) + (b * a.pow(k.suc) - b.pow(k.suc.suc)) =
                a.pow(k.suc.suc) + (-(b * a.pow(k.suc)) + (b * a.pow(k.suc) + -b.pow(k.suc.suc)))
            -(b * a.pow(k.suc)) + (b * a.pow(k.suc) + -b.pow(k.suc.suc)) =
                (-(b * a.pow(k.suc)) + b * a.pow(k.suc)) + -b.pow(k.suc.suc)
            -(b * a.pow(k.suc)) + b * a.pow(k.suc) = R.0
            a.pow(k.suc.suc) + (R.0 + -b.pow(k.suc.suc)) = a.pow(k.suc.suc) - b.pow(k.suc.suc)
            (a.pow(k.suc.suc) - b * a.pow(k.suc)) + (b * a.pow(k.suc) - b.pow(k.suc.suc)) =
                a.pow(k.suc.suc) - b.pow(k.suc.suc)
            (a - b) * partial(geom_sum_term[R](a, b, k.suc), k.suc.suc) =
                a.pow(k.suc.suc) - b.pow(k.suc.suc)
            statement(k.suc)
        }
    }

    statement(Nat.0) and forall(k: Nat) {
        statement(k) implies statement(k.suc)
    }
    alt_induction(statement)
    forall(k: Nat) { statement(k) }
    statement(n)
}

/// The classical geometric sum identity:
/// `(a - b) * (a^(n-1) + a^(n-2) b + ... + b^(n-1)) = a^n - b^n`.
theorem geom_sum_classical[R: CommRing](a: R, b: R, n: Nat) {
    (a - b) * partial(geom_sum_term[R](a, b, n - Nat.1), n) = a.pow(n) - b.pow(n)
} by {
    geom_sum(a, b, n - Nat.1)
    (a - b) * partial(geom_sum_term[R](a, b, n - Nat.1), (n - Nat.1).suc) =
        a.pow((n - Nat.1).suc) - b.pow((n - Nat.1).suc)
    if n = Nat.0 {
        n - Nat.1 = Nat.0
        partial_zero(geom_sum_term[R](a, b, n - Nat.1))
        partial(geom_sum_term[R](a, b, n - Nat.1), Nat.0) = R.0
        n = Nat.0
        partial(geom_sum_term[R](a, b, n - Nat.1), n) = R.0
        (a - b) * partial(geom_sum_term[R](a, b, n - Nat.1), n) = (a - b) * R.0
        (a - b) * R.0 = R.0
        a.pow(n) = a.pow(Nat.0)
        a.pow(Nat.0) = R.1
        b.pow(n) = b.pow(Nat.0)
        b.pow(Nat.0) = R.1
        a.pow(n) - b.pow(n) = R.1 - R.1
        R.1 - R.1 = R.0
        (a - b) * partial(geom_sum_term[R](a, b, n - Nat.1), n) = a.pow(n) - b.pow(n)
    } else {
        zero_or_suc(n)
        let np: Nat satisfy {
            n = np.suc
        }
        n = np.suc
        n - Nat.1 = np
        (n - Nat.1).suc = n
        partial(geom_sum_term[R](a, b, n - Nat.1), (n - Nat.1).suc) =
            partial(geom_sum_term[R](a, b, n - Nat.1), n)
        a.pow((n - Nat.1).suc) = a.pow(n)
        b.pow((n - Nat.1).suc) = b.pow(n)
        (a - b) * partial(geom_sum_term[R](a, b, n - Nat.1), n) = a.pow(n) - b.pow(n)
    }
}
