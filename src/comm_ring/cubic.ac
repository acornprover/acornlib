from comm_ring.base import CommRing
from nat import Nat

/// The square of a commutative-ring element as a second power.
theorem pow_two_mul_comm_ring[R: CommRing](x: R) {
    x.pow(Nat.2) = x * x
} by {
}

/// The cube of a commutative-ring element as a third power.
theorem pow_three_mul_comm_ring[R: CommRing](x: R) {
    x.pow(Nat.3) = x * x * x
} by {
}

/// Multiplication by two, written as `1 + 1`, duplicates an element.
theorem two_mul_comm_ring[R: CommRing](x: R) {
    (R.1 + R.1) * x = x + x
} by {
}

/// Multiplication by three, written as `1 + 1 + 1`, triples an element.
theorem three_mul_comm_ring[R: CommRing](x: R) {
    (R.1 + R.1 + R.1) * x = x + (x + x)
} by {
    two_mul_comm_ring[R](x)
}

/// Expansion of `(u + v)^2` in a commutative ring.
theorem square_add_expand_comm_ring[R: CommRing](u: R, v: R) {
    (u + v) * (u + v) = u * u + (R.1 + R.1) * u * v + v * v
} by {
    (u + v) * v = u * v + v * v
    (u * u + u * v) + (u * v + v * v) = u * u + (u * v + u * v) + v * v
}

/// Expansion of `(u + v)^2` with the two mixed terms left explicit.
theorem square_add_expand_raw_comm_ring[R: CommRing](u: R, v: R) {
    (u + v) * (u + v) = u * u + (u * v + u * v) + v * v
} by {
    (u + v) * v = u * v + v * v
    (u * u + u * v) + (u * v + v * v) = u * u + (u * v + u * v) + v * v
}

/// Power-form expansion of `(u + v)^2` with the two mixed terms left explicit.
theorem square_add_pow_raw_comm_ring[R: CommRing](u: R, v: R) {
    (u + v).pow(Nat.2) = u * u + (u * v + u * v) + v * v
} by {
    pow_two_mul_comm_ring[R](u + v)
    square_add_expand_raw_comm_ring[R](u, v)
}

/// Power-form expansion of `(u + v)^2` with collected mixed term.
theorem square_add_pow_comm_ring[R: CommRing](u: R, v: R) {
    (u + v).pow(Nat.2) = u.pow(Nat.2) + (R.1 + R.1) * u * v + v.pow(Nat.2)
} by {
    pow_two_mul_comm_ring[R](u + v)
    pow_two_mul_comm_ring[R](u)
    pow_two_mul_comm_ring[R](v)
    square_add_expand_comm_ring[R](u, v)
}

/// Raw left-associated expansion of a cubic product.
theorem left_cube_raw_comm_ring[R: CommRing](u: R, v: R) {
    (u * u + (u * v + u * v) + v * v) * (u + v) =
    ((u * u) * u + (u * u) * v) +
    (((u * v) * u + (u * v) * u) +
     ((u * v) * v + (u * v) * v) +
     ((v * v) * u + (v * v) * v))
} by {
    (u * v + u * v) * (u + v) = ((u * v) * u + (u * v) * u) + ((u * v) * v + (u * v) * v)
    (u * u + (u * v + u * v)) * (u + v) = ((u * u) * u + (u * u) * v) + (((u * v) * u + (u * v) * u) + ((u * v) * v + (u * v) * v))
}

/// Three equal pairs regroup into two triples.
theorem regroup_three_pairs_comm_ring[R: CommRing](a: R, b: R) {
    (a + b) + ((a + b) + (a + b)) = (a + (a + a)) + (b + (b + b))
} by {
    (a + b) + ((a + b) + (a + b)) = a + (b + ((a + b) + (a + b)))
    b + ((a + b) + (a + b)) = b + (a + (b + (a + b)))
    b + (a + (b + (a + b))) = a + (b + (b + (a + b)))
    b + (b + (a + b)) = b + (b + (b + a))
    b + (b + (b + a)) = a + (b + (b + b))
    a + (a + (a + (b + (b + b)))) = (a + (a + a)) + (b + (b + b))
}

/// Left-associated form of the three-pair regrouping identity.
theorem regroup_three_pairs_left_assoc_comm_ring[R: CommRing](a: R, b: R) {
    (a + b) + (a + b) + (a + b) = (a + (a + a)) + (b + (b + b))
} by {
    regroup_three_pairs_comm_ring[R](a, b)
}

/// A triple coefficient times a mixed product splits into three mixed products.
theorem three_uv_raw_comm_ring[R: CommRing](u: R, v: R) {
    (((R.1 + R.1 + R.1) * u * v) * (u + v)) =
    ((u * v) * u + ((u * v) * u + (u * v) * u)) +
    ((u * v) * v + ((u * v) * v + (u * v) * v))
} by {
    ((R.1 + R.1 + R.1) * u * v) * (u + v) = (u * v + (u * v + u * v)) * (u + v)
    (u * v + u * v) * (u + v) = ((u * v) * u + (u * v) * v) + ((u * v) * u + (u * v) * v)
    regroup_three_pairs_comm_ring[R]((u * v) * u, (u * v) * v)
}

/// Six summands regroup into outer terms and two triples.
theorem add_regroup_six_comm_ring[R: CommRing](x: R, y: R, a: R, b: R) {
    (x + a) + (((a + a) + (b + b)) + (b + y)) =
    x + y + ((a + (a + a)) + (b + (b + b)))
} by {
    (b + b) + (b + y) = (b + (b + b)) + y
    (a + (a + a)) + ((b + (b + b)) + y) = y + ((a + (a + a)) + (b + (b + b)))
}

/// Raw cubic terms regroup into pure cubes and mixed triples.
theorem raw_cube_regroup_comm_ring[R: CommRing](u: R, v: R) {
    ((u * u) * u + (u * u) * v) +
    (((u * v) * u + (u * v) * u) +
     ((u * v) * v + (u * v) * v) +
     ((v * v) * u + (v * v) * v)) =
    u * u * u + v * v * v + (((R.1 + R.1 + R.1) * u * v) * (u + v))
} by {
    three_uv_raw_comm_ring[R](u, v)
    (u * u) * u = u * u * u
    (u * u) * v = (u * v) * u
    (v * v) * u = (u * v) * v
    (v * v) * v = v * v * v
    let a: R = (u * v) * u
    let b: R = (u * v) * v
    let x: R = u * u * u
    let y: R = v * v * v
    add_regroup_six_comm_ring[R](x, y, a, b)
}

/// The expanded square times a binomial gives the Cardano cubic form.
theorem cube_add_expanded_square_comm_ring[R: CommRing](u: R, v: R) {
    (u * u + (u * v + u * v) + v * v) * (u + v) =
    u * u * u + v * v * v + (((R.1 + R.1 + R.1) * u * v) * (u + v))
} by {
    left_cube_raw_comm_ring[R](u, v)
    raw_cube_regroup_comm_ring[R](u, v)
}

/// Product-form Cardano expansion in a commutative ring.
theorem cube_add_cardano_mul_comm_ring[R: CommRing](u: R, v: R) {
    (u + v) * (u + v) * (u + v) =
    u * u * u + v * v * v + (((R.1 + R.1 + R.1) * u * v) * (u + v))
} by {
    square_add_expand_raw_comm_ring[R](u, v)
    cube_add_expanded_square_comm_ring[R](u, v)
}

/// Power-form Cardano expansion in a commutative ring.
theorem cube_add_cardano_comm_ring[R: CommRing](u: R, v: R) {
    (u + v).pow(Nat.3) =
    u.pow(Nat.3) + v.pow(Nat.3) + (((R.1 + R.1 + R.1) * u * v) * (u + v))
} by {
    pow_three_mul_comm_ring[R](u + v)
    pow_three_mul_comm_ring[R](u)
    pow_three_mul_comm_ring[R](v)
    cube_add_cardano_mul_comm_ring[R](u, v)
}

/// Cardano's depressed-cubic substitution, as a reusable algebraic helper.
theorem cardano_depressed_cubic_substitution_comm_ring[R: CommRing](u: R, v: R, p: R, q: R) {
    ((R.1 + R.1 + R.1) * u * v + p = R.0 and
     u.pow(Nat.3) + v.pow(Nat.3) + q = R.0) implies
    (u + v).pow(Nat.3) + p * (u + v) + q = R.0
} by {
    let t: R = (R.1 + R.1 + R.1) * u * v
    let x: R = u + v
    (u + v).pow(Nat.3) = u.pow(Nat.3) + v.pow(Nat.3) + (t * x)
    (u + v).pow(Nat.3) + p * x + q = (u.pow(Nat.3) + v.pow(Nat.3) + t * x) + p * x + q
    (t + p) * x = R.0
}

/// Four-term expansion of `(u + v)^2`, without collecting like terms.
theorem square_four_terms_comm_ring[R: CommRing](u: R, v: R) {
    (u + v) * (u + v) = u * u + u * v + v * u + v * v
} by {
}

/// Multiplication distributes across a four-term sum in a commutative ring.
theorem four_terms_mul_comm_ring[R: CommRing](a: R, b: R, c: R, d: R, x: R) {
    (a + b + c + d) * x = a * x + b * x + c * x + d * x
} by {
    ((a + b) * x + c * x) + d * x = ((a * x + b * x) + c * x) + d * x
    ((a * x + b * x) + c * x) + d * x = a * x + b * x + c * x + d * x
}

/// Factor the six mixed terms that occur in `(u + v)^3`.
theorem triple_uv_factor_comm_ring[R: CommRing](u: R, v: R) {
    ((R.1 + R.1 + R.1) * u * v) * (u + v) =
    u * u * v + u * v * u + v * u * u + u * v * v + v * u * v + v * v * u
} by {
    (R.1 + R.1 + R.1) * u = R.1 * u + R.1 * u + R.1 * u
    R.1 * u = u
    (R.1 + R.1 + R.1) * u = u + u + u
    (R.1 + R.1 + R.1) * u * v = (u + u + u) * v
    (u + u + u) * v = u * v + u * v + u * v
    (R.1 + R.1 + R.1) * u * v = u * v + u * v + u * v
    ((R.1 + R.1 + R.1) * u * v) * (u + v) = (u * v + u * v + u * v) * (u + v)
    (u * v + u * v + u * v) * (u + v) =
        (u * v + u * v + u * v) * u + (u * v + u * v + u * v) * v
    (u * v + u * v + u * v) * u = (u * v) * u + (u * v) * u + (u * v) * u
    (u * v + u * v + u * v) * v = (u * v) * v + (u * v) * v + (u * v) * v
    (u * v) * u = u * v * u
    (u * v) * u = u * u * v
    (u * v) * u = v * u * u
    (u * v) * v = u * v * v
    (u * v) * v = v * u * v
    (u * v) * v = v * v * u
    (u * v + u * v + u * v) * (u + v) =
        u * u * v + u * v * u + v * u * u + u * v * v + v * u * v + v * v * u
}
