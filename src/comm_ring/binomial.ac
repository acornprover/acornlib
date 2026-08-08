from comm_ring.base import CommRing
from nat import Nat, semiring_zero_pow
from combinatorics import binom, binom_absorption_suc, pascal_suc_unbounded
from nat import from_nat, from_nat_add, from_nat_mul
from list import partial, partial_drop_first, partial_pointwise_eq, partial_scalar_mul
from algebra.add_semigroup import add_fn
from algebra.semigroup import mul_fn
from data.basic.functions import compose
from algebra.ring.ring import alternating_sign, alternating_sign_eq_neg_one_pow,
    alternating_sign_parity, alternating_sign_suc, mul_neg_left, mul_neg_one_right,
    mul_zero_left
from algebra.add_comm_group import sub_add_sub, sub_eq_zero_imp_eq

numerals Nat

/// The binomial theorem for commutative rings.
///
/// This file proves that (a + b)^n equals the sum of binomial terms for elements
/// of a commutative ring, generalizing the binomial theorem from natural numbers.

/// A term in the binomial expansion of (a + b)^n.
/// Represents n.binom(k) * a^k * b^(n-k).
define binomial_term[R: CommRing](a: R, b: R, n: Nat, k: Nat) -> R {
    from_nat[R](n.binom(k)) * a.pow(k) * b.pow(n - k)
}

/// The base binomial term at k=0 is the pure b-power.
theorem binomial_term_zero_power[R: CommRing](a: R, b: R, n: Nat) {
    binomial_term[R](a, b, n, 0) = b.pow(n)
} by {
    binomial_term[R](a, b, n, 0) = from_nat[R](n.binom(0)) * a.pow(0) * b.pow(n)
    binomial_term[R](a, b, n, 0) = from_nat[R](1) * R.1 * b.pow(n)
    binomial_term[R](a, b, n, 0) = b.pow(n)
}

/// The top binomial term at k=n is the pure a-power.
theorem binomial_term_top_power[R: CommRing](a: R, b: R, n: Nat) {
    binomial_term[R](a, b, n, n) = a.pow(n)
} by {
    n - n = 0
    binomial_term[R](a, b, n, n) = from_nat[R](n.binom(n)) * a.pow(n) * b.pow(n - n)
    binomial_term[R](a, b, n, n) = from_nat[R](n.binom(n)) * a.pow(n) * b.pow(0)
    binomial_term[R](a, b, n, n) = from_nat[R](1) * a.pow(n) * R.1
    binomial_term[R](a, b, n, n) = a.pow(n)
}

/// The unique zeroth-row binomial term is one.
theorem binomial_term_zero_zero[R: CommRing](a: R, b: R) {
    binomial_term[R](a, b, 0, 0) = R.1
} by {
    binomial_term_zero_power[R](a, b, 0)
}

/// The k=0 term in the successor row is b times the previous pure b-power.
theorem binomial_term_zero_succ_power[R: CommRing](a: R, b: R, m: Nat) {
    binomial_term[R](a, b, m.suc, 0) = b * b.pow(m)
} by {
    binomial_term_zero_power[R](a, b, m.suc)
}

/// The top term in the successor row is a times the previous pure a-power.
theorem binomial_term_top_succ_power[R: CommRing](a: R, b: R, m: Nat) {
    binomial_term[R](a, b, m.suc, m.suc) = a * a.pow(m)
} by {
    binomial_term_top_power[R](a, b, m.suc)
}

/// The boundary term at k=0 for the binomial expansion.
theorem binomial_term_zero[R: CommRing](a: R, b: R, m: Nat) {
    binomial_term[R](a, b, m.suc, 0) = b * binomial_term[R](a, b, m, 0)
} by {
    // Expand both sides

}

/// The boundary term at k=m+1 for the binomial expansion.
theorem binomial_term_top[R: CommRing](a: R, b: R, m: Nat) {
    binomial_term[R](a, b, m.suc, m.suc) = a * binomial_term[R](a, b, m, m)
} by {
    // Expand both sides
    binomial_term[R](a, b, m, m) = from_nat[R](1) * a.pow(m) * R.1

}

/// Recursive relation for binomial terms: each term for n+1 equals a times the shifted
/// term for n plus b times the corresponding term for n.
theorem binomial_term_recurrence[R: CommRing](a: R, b: R, m: Nat, k: Nat) {
    0 < k and k <= m implies
    binomial_term[R](a, b, m.suc, k) =
    a * binomial_term[R](a, b, m, k - 1) + b * binomial_term[R](a, b, m, k)
} by {
    // Establish k_pred for k - 1
    let (k_pred: Nat) satisfy { k_pred.suc = k }
    k_pred <= m

    // Expand LHS: binomial_term(a, b, m.suc, k)
    binomial_term[R](a, b, m.suc, k) = from_nat[R](m.suc.binom(k)) * a.pow(k) * b.pow(m.suc - k)

    // Use Pascal's identity
    m.suc.binom(k) = m.binom(k - 1) + m.binom(k)
    from_nat[R](m.suc.binom(k)) = from_nat[R](m.binom(k - 1) + m.binom(k))
    from_nat[R](m.suc.binom(k)) = from_nat[R](m.binom(k - 1)) + from_nat[R](m.binom(k))

    // Distribute
    binomial_term[R](a, b, m.suc, k) = (from_nat[R](m.binom(k - 1)) + from_nat[R](m.binom(k))) * a.pow(k) * b.pow(m.suc - k)
    binomial_term[R](a, b, m.suc, k) = from_nat[R](m.binom(k - 1)) * a.pow(k) * b.pow(m.suc - k) + from_nat[R](m.binom(k)) * a.pow(k) * b.pow(m.suc - k)

    // First term: a * binomial_term(a, b, m, k - 1)
    a.pow(k) = a * a.pow(k_pred)
    m - k_pred + k_pred = m
    m - k_pred + k = m.suc
    m - k_pred = m.suc - k
    from_nat[R](m.binom(k_pred)) * a.pow(k) * b.pow(m.suc - k) = from_nat[R](m.binom(k_pred)) * a * a.pow(k_pred) * b.pow(m - k_pred)
    from_nat[R](m.binom(k_pred)) * a.pow(k) * b.pow(m.suc - k) = a * (from_nat[R](m.binom(k_pred)) * a.pow(k_pred) * b.pow(m - k_pred))

    // Second term: b * binomial_term(a, b, m, k)
    m - k + k = m
    (m - k).suc + k = (m - k + k).suc
    m - k + 1 + k = m.suc
    b.pow(m.suc - k) = b.pow(m - k + 1)
    b.pow(m - k + 1) = b * b.pow(m - k)
    from_nat[R](m.binom(k)) * a.pow(k) * b.pow(m.suc - k) = from_nat[R](m.binom(k)) * a.pow(k) * b * b.pow(m - k)
    from_nat[R](m.binom(k)) * a.pow(k) * b.pow(m.suc - k) = b * (from_nat[R](m.binom(k)) * a.pow(k) * b.pow(m - k))
}

/// Alternative form of the binomial term recurrence, avoiding subtraction by using k.suc.
theorem alt_binomial_term_recurrence[R: CommRing](a: R, b: R, m: Nat, k: Nat) {
    k < m implies
    binomial_term[R](a, b, m.suc, k.suc) =
    a * binomial_term[R](a, b, m, k) + b * binomial_term[R](a, b, m, k.suc)
} by {
    // We have k < m, which gives us k.suc <= m and 0 < k.suc
    binomial_term[R](a, b, m.suc, k.suc) = a * binomial_term[R](a, b, m, k.suc - 1) + b * binomial_term[R](a, b, m, k.suc)
}

/// Successor-index recurrence wrapper for the interior of a binomial row.
theorem binomial_term_recurrence_succ_index[R: CommRing](a: R, b: R, m: Nat, k: Nat) {
    k < m implies
    binomial_term[R](a, b, m.suc, k.suc) =
    a * binomial_term[R](a, b, m, k) + b * binomial_term[R](a, b, m, k.suc)
} by {
    if k < m {
        alt_binomial_term_recurrence[R](a, b, m, k)
    }
}

/// Helper: the middle partial sum splits into two parts using the recurrence.
theorem binomial_middle_sum[R: CommRing](a: R, b: R, m: Nat) {
    partial(compose(binomial_term[R](a, b, m.suc), Nat.suc), m) =
        a * partial(binomial_term[R](a, b, m), m) + b * partial(compose(binomial_term[R](a, b, m), Nat.suc), m)
} by {
    // Define helper functions for clarity
    define f(k: Nat) -> R { binomial_term[R](a, b, m, k) }
    define g(k: Nat) -> R { compose(binomial_term[R](a, b, m), Nat.suc)(k) }
    define h(k: Nat) -> R { compose(binomial_term[R](a, b, m.suc), Nat.suc)(k) }

    // Show pointwise equality: for each k < m, h(k) = a * f(k) + b * g(k)
    forall(k: Nat) {
        if k < m {
            // h(k) = binomial_term(a, b, m.suc, k.suc)

            // Use alt_binomial_term_recurrence
            binomial_term[R](a, b, m.suc, k.suc) = a * binomial_term[R](a, b, m, k) + b * binomial_term[R](a, b, m, k.suc)

            // Simplify RHS
            h(k) = mul_fn(a, f)(k) + mul_fn(b, g)(k)
            h(k) = add_fn(mul_fn(a, f), mul_fn(b, g), k)
        }
    }

    // Use partial_pointwise_eq
    partial(h, m) = partial(add_fn(mul_fn(a, f), mul_fn(b, g)), m)

    // Use partial_add to split the sum

    // Use partial_scalar_mul to factor out the scalars
}

/// Distributing (a + b) across a partial sum of binomial terms gives the next row.
theorem binomial_distribution[R: CommRing](a: R, b: R, m: Nat) {
    (a + b) * partial(binomial_term[R](a, b, m), m.suc) = partial(binomial_term[R](a, b, m.suc), m.suc.suc)
} by {
    // LHS: Expand using distributivity
    (a + b) * partial(binomial_term[R](a, b, m), m.suc) = a * partial(binomial_term[R](a, b, m), m.suc) + b * partial(binomial_term[R](a, b, m), m.suc)

    // RHS: Peel off the last term using partial_split_last
    partial(binomial_term[R](a, b, m.suc), m.suc.suc) = partial(binomial_term[R](a, b, m.suc), m.suc) + binomial_term[R](a, b, m.suc, m.suc)

    // Simplify the last term
    binomial_term[R](a, b, m.suc, m.suc) = a * binomial_term[R](a, b, m, m)

    // Now peel off the first term using partial_shift_suc
    partial(binomial_term[R](a, b, m.suc), m.suc) = binomial_term[R](a, b, m.suc, 0) + partial(compose(binomial_term[R](a, b, m.suc), Nat.suc), m)

    // Simplify the first term
    binomial_term[R](a, b, m.suc, 0) = b * binomial_term[R](a, b, m, 0)

    // Use binomial_middle_sum to split the middle partial sum
    partial(compose(binomial_term[R](a, b, m.suc), Nat.suc), m) = a * partial(binomial_term[R](a, b, m), m) + b * partial(compose(binomial_term[R](a, b, m), Nat.suc), m)

    // Build up the RHS step by step
    // Start with: RHS = first + middle + last
    let rhs_part1 = b * binomial_term[R](a, b, m, 0) + partial(compose(binomial_term[R](a, b, m.suc), Nat.suc), m)

    // Expand the middle part using binomial_middle_sum
    rhs_part1 = b * binomial_term[R](a, b, m, 0) + a * partial(binomial_term[R](a, b, m), m) + b * partial(compose(binomial_term[R](a, b, m), Nat.suc), m)

    // Combine the 'b' terms using partial_shift_suc
    b * binomial_term[R](a, b, m, 0) + b * partial(compose(binomial_term[R](a, b, m), Nat.suc), m) = b * partial(binomial_term[R](a, b, m), m.suc)

    // So rhs_part1 = b * partial(...) + a * partial(..., m)
    rhs_part1 = b * partial(binomial_term[R](a, b, m), m.suc) + a * partial(binomial_term[R](a, b, m), m)

    // Therefore the full RHS is
    partial(binomial_term[R](a, b, m.suc), m.suc.suc) = b * partial(binomial_term[R](a, b, m), m.suc) + a * partial(binomial_term[R](a, b, m), m) + a * binomial_term[R](a, b, m, m)

    // Combine the 'a' terms using partial_split_last
    a * partial(binomial_term[R](a, b, m), m) + a * binomial_term[R](a, b, m, m) = a * partial(binomial_term[R](a, b, m), m.suc)

    // Final result
}

/// The binomial theorem for commutative rings: (a + b)^n equals the sum of binomial terms.
theorem binomial[R: CommRing](a: R, b: R, n: Nat) {
    (a + b).pow(n) = partial(binomial_term[R](a, b, n), n.suc)
} by {
    define f(x: Nat) -> Bool {
        (a + b).pow(x) = partial(binomial_term[R](a, b, x), x.suc)
    }

    // Base case: n = 0
    (a + b).pow(0) = R.1
    binomial_term[R](a, b, 0, 0) = from_nat[R](0.binom(0)) * a.pow(0) * b.pow(0)
    binomial_term[R](a, b, 0, 0) = from_nat[R](1) * R.1 * R.1
    binomial_term[R](a, b, 0, 0) = R.1
    partial(binomial_term[R](a, b, 0), 0.suc) = binomial_term[R](a, b, 0, 0)
    partial(binomial_term[R](a, b, 0), 0.suc) = R.1
    f(0)
    f(Nat.0)
    f(Nat.zero)

    // Inductive step
    forall(m: Nat) {
        if f(m) {
            // Induction hypothesis: (a + b)^m = partial(binomial_term(a, b, m), m.suc)

            // LHS: (a + b)^(m+1) = (a + b) * (a + b)^m
            partial[R](binomial_term(a, b, m), m.suc) = (a + b).pow(m)

            // Apply binomial_distribution to complete the inductive step
            (a + b).pow(m.suc) = partial(binomial_term[R](a, b, m.suc), m.suc.suc)
            f(m.suc)
        }
    }
    f(n)
}

/// The alternating sum of the binomial coefficients in a positive row is zero.
theorem alternating_binomial_row_sum[R: CommRing](n: Nat) {
    n > Nat.0 implies
        partial(binomial_term[R](-R.1, R.1, n), n.suc) = R.0
} by {
    if n > Nat.0 {
        binomial[R](-R.1, R.1, n)
        partial(binomial_term[R](-R.1, R.1, n), n.suc) =
            (-R.1 + R.1).pow(n)
        -R.1 + R.1 = R.0
        partial(binomial_term[R](-R.1, R.1, n), n.suc) = R.0.pow(n)
        semiring_zero_pow[R](n)
        R.0.pow(n) = R.0
        partial(binomial_term[R](-R.1, R.1, n), n.suc) = R.0
    }
}

/// The binomial coefficient in an even position, and zero in an odd position.
define even_binomial_row_term[R: CommRing](n: Nat, k: Nat) -> R {
    if Nat.2.divides(k) {
        from_nat[R](n.binom(k))
    } else {
        R.0
    }
}

/// The binomial coefficient in an odd position, and zero in an even position.
define odd_binomial_row_term[R: CommRing](n: Nat, k: Nat) -> R {
    if Nat.2.divides(k) {
        R.0
    } else {
        from_nat[R](n.binom(k))
    }
}

/// An alternating binomial term is its even part minus its odd part.
theorem alternating_binomial_term_parity[R: CommRing](n: Nat, k: Nat) {
    binomial_term[R](-R.1, R.1, n, k) =
        even_binomial_row_term[R](n, k) - odd_binomial_row_term[R](n, k)
} by {
    let coefficient = from_nat[R](n.binom(k))
    binomial_term[R](-R.1, R.1, n, k) =
        coefficient * (-R.1).pow(k) * R.1.pow(n - k)
    R.1.pow(n - k) = R.1
    binomial_term[R](-R.1, R.1, n, k) = coefficient * (-R.1).pow(k)
    alternating_sign_eq_neg_one_pow[R](k)
    alternating_sign[R](k) = (-R.1).pow(k)
    alternating_sign_parity[R](k)

    if Nat.2.divides(k) {
        alternating_sign[R](k) = R.1
        (-R.1).pow(k) = R.1
        even_binomial_row_term[R](n, k) = coefficient
        odd_binomial_row_term[R](n, k) = R.0
        binomial_term[R](-R.1, R.1, n, k) = coefficient
        even_binomial_row_term[R](n, k) - odd_binomial_row_term[R](n, k) =
            coefficient - R.0
        coefficient - R.0 = coefficient
        even_binomial_row_term[R](n, k) - odd_binomial_row_term[R](n, k) = coefficient
        binomial_term[R](-R.1, R.1, n, k) =
            even_binomial_row_term[R](n, k) - odd_binomial_row_term[R](n, k)
    } else {
        alternating_sign[R](k) = -R.1
        (-R.1).pow(k) = -R.1
        even_binomial_row_term[R](n, k) = R.0
        odd_binomial_row_term[R](n, k) = coefficient
        mul_neg_one_right[R](coefficient)
        coefficient * -R.1 = -coefficient
        binomial_term[R](-R.1, R.1, n, k) = -coefficient
        even_binomial_row_term[R](n, k) - odd_binomial_row_term[R](n, k) = -coefficient
        binomial_term[R](-R.1, R.1, n, k) =
            even_binomial_row_term[R](n, k) - odd_binomial_row_term[R](n, k)
    }
    binomial_term[R](-R.1, R.1, n, k) =
        even_binomial_row_term[R](n, k) - odd_binomial_row_term[R](n, k)
}

/// Every alternating partial row is the even partial row minus the odd partial row.
theorem alternating_binomial_partial_decomposition[R: CommRing](n: Nat, m: Nat) {
    partial(binomial_term[R](-R.1, R.1, n), m) =
        partial(even_binomial_row_term[R](n), m) -
        partial(odd_binomial_row_term[R](n), m)
} by {
    define p(j: Nat) -> Bool {
        partial(binomial_term[R](-R.1, R.1, n), j) =
            partial(even_binomial_row_term[R](n), j) -
            partial(odd_binomial_row_term[R](n), j)
    }

    partial(binomial_term[R](-R.1, R.1, n), Nat.0) = R.0
    partial(even_binomial_row_term[R](n), Nat.0) = R.0
    partial(odd_binomial_row_term[R](n), Nat.0) = R.0
    R.0 - R.0 = R.0
    p(Nat.0)
    forall(j: Nat) {
        if p(j) {
            let alternating_partial = partial(binomial_term[R](-R.1, R.1, n), j)
            let even_partial = partial(even_binomial_row_term[R](n), j)
            let odd_partial = partial(odd_binomial_row_term[R](n), j)
            let alternating_term = binomial_term[R](-R.1, R.1, n, j)
            let even_term = even_binomial_row_term[R](n, j)
            let odd_term = odd_binomial_row_term[R](n, j)

            alternating_partial = even_partial - odd_partial
            alternating_binomial_term_parity[R](n, j)
            alternating_term = even_term - odd_term
            partial(binomial_term[R](-R.1, R.1, n), j.suc) =
                alternating_partial + alternating_term
            partial(even_binomial_row_term[R](n), j.suc) =
                even_partial + even_term
            partial(odd_binomial_row_term[R](n), j.suc) =
                odd_partial + odd_term
            partial(binomial_term[R](-R.1, R.1, n), j.suc) =
                (even_partial - odd_partial) + (even_term - odd_term)
            sub_add_sub[R](even_partial, odd_partial, even_term, odd_term)
            (even_partial - odd_partial) + (even_term - odd_term) =
                (even_partial + even_term) - (odd_partial + odd_term)
            partial(binomial_term[R](-R.1, R.1, n), j.suc) =
                partial(even_binomial_row_term[R](n), j.suc) -
                partial(odd_binomial_row_term[R](n), j.suc)
            p(j.suc)
        }
    }
    p(Nat.0) and forall(j: Nat) {
        p(j) implies p(j.suc)
    }
    Nat.induction(p)
    p(m)
}

/// The even- and odd-index binomial sums in a positive row are equal.
theorem even_odd_binomial_row_sums_equal[R: CommRing](n: Nat) {
    n > Nat.0 implies
        partial(even_binomial_row_term[R](n), n.suc) =
        partial(odd_binomial_row_term[R](n), n.suc)
} by {
    if n > Nat.0 {
        alternating_binomial_row_sum[R](n)
        alternating_binomial_partial_decomposition[R](n, n.suc)
        partial(even_binomial_row_term[R](n), n.suc) -
            partial(odd_binomial_row_term[R](n), n.suc) = R.0
        sub_eq_zero_imp_eq[R](
            partial(even_binomial_row_term[R](n), n.suc),
            partial(odd_binomial_row_term[R](n), n.suc))
        partial(even_binomial_row_term[R](n), n.suc) =
            partial(odd_binomial_row_term[R](n), n.suc)
    }
}

/// An alternating binomial term is its coefficient multiplied by its sign.
theorem alternating_binomial_term_eq_sign_mul[R: CommRing](n: Nat, k: Nat) {
    binomial_term[R](-R.1, R.1, n, k) =
        alternating_sign[R](k) * from_nat[R](n.binom(k))
} by {
    let coefficient = from_nat[R](n.binom(k))
    binomial_term[R](-R.1, R.1, n, k) =
        coefficient * (-R.1).pow(k) * R.1.pow(n - k)
    R.1.pow(n - k) = R.1
    binomial_term[R](-R.1, R.1, n, k) = coefficient * (-R.1).pow(k)
    alternating_sign_eq_neg_one_pow[R](k)
    alternating_sign[R](k) = (-R.1).pow(k)
    coefficient * (-R.1).pow(k) = alternating_sign[R](k) * coefficient
    binomial_term[R](-R.1, R.1, n, k) = alternating_sign[R](k) * coefficient
}

/// The first `m + 1` alternating terms of row `n + 1` equal the signed
/// coefficient in position `m` of row `n`.
theorem alternating_binomial_partial_row[R: CommRing](n: Nat, m: Nat) {
    partial(binomial_term[R](-R.1, R.1, n.suc), m.suc) =
        alternating_sign[R](m) * from_nat[R](n.binom(m))
} by {
    define p(j: Nat) -> Bool {
        partial(binomial_term[R](-R.1, R.1, n.suc), j.suc) =
            alternating_sign[R](j) * from_nat[R](n.binom(j))
    }

    alternating_binomial_term_eq_sign_mul[R](n.suc, Nat.0)
    partial(binomial_term[R](-R.1, R.1, n.suc), Nat.1) =
        binomial_term[R](-R.1, R.1, n.suc, Nat.0)
    alternating_sign[R](Nat.0) = R.1
    n.suc.binom(Nat.0) = Nat.1
    n.binom(Nat.0) = Nat.1
    from_nat[R](Nat.1) = R.1
    p(Nat.0)

    forall(j: Nat) {
        if p(j) {
            let sign = alternating_sign[R](j)
            let coefficient = from_nat[R](n.binom(j))
            let next_coefficient = from_nat[R](n.binom(j.suc))

            partial(binomial_term[R](-R.1, R.1, n.suc), j.suc) =
                sign * coefficient
            partial(binomial_term[R](-R.1, R.1, n.suc), j.suc.suc) =
                partial(binomial_term[R](-R.1, R.1, n.suc), j.suc) +
                binomial_term[R](-R.1, R.1, n.suc, j.suc)

            pascal_suc_unbounded(n, j)
            n.suc.binom(j.suc) = n.binom(j) + n.binom(j.suc)
            from_nat_add[R](n.binom(j), n.binom(j.suc))
            from_nat[R](n.suc.binom(j.suc)) = coefficient + next_coefficient
            alternating_binomial_term_eq_sign_mul[R](n.suc, j.suc)
            alternating_sign_suc[R](j)
            alternating_sign[R](j.suc) = -sign
            binomial_term[R](-R.1, R.1, n.suc, j.suc) =
                (-sign) * (coefficient + next_coefficient)

            partial(binomial_term[R](-R.1, R.1, n.suc), j.suc.suc) =
                sign * coefficient + (-sign) * (coefficient + next_coefficient)
            (-sign) * (coefficient + next_coefficient) =
                (-sign) * coefficient + (-sign) * next_coefficient
            mul_neg_left[R](sign, coefficient)
            (-sign) * coefficient = -(sign * coefficient)
            sign * coefficient + ((-sign) * coefficient + (-sign) * next_coefficient) =
                (sign * coefficient + -(sign * coefficient)) +
                (-sign) * next_coefficient
            sign * coefficient + -(sign * coefficient) = R.0
            (sign * coefficient + -(sign * coefficient)) +
                (-sign) * next_coefficient = (-sign) * next_coefficient
            partial(binomial_term[R](-R.1, R.1, n.suc), j.suc.suc) =
                alternating_sign[R](j.suc) * next_coefficient
            p(j.suc)
        }
    }
    p(Nat.0) and forall(j: Nat) {
        p(j) implies p(j.suc)
    }
    Nat.induction(p)
    p(m)
}

/// The `k`th formal derivative term in the binomial expansion of `(a + b)^n`.
define binomial_derivative_term[R: CommRing](a: R, b: R, n: Nat, k: Nat) -> R {
    from_nat[R](k) * from_nat[R](n.binom(k)) *
        a.pow(k - Nat.1) * b.pow(n - k)
}

/// The zeroth formal derivative term is zero.
theorem binomial_derivative_term_zero[R: CommRing](a: R, b: R, n: Nat) {
    binomial_derivative_term[R](a, b, n, Nat.0) = R.0
} by {
    binomial_derivative_term[R](a, b, n, Nat.0) =
        from_nat[R](Nat.0) * from_nat[R](n.binom(Nat.0)) *
            a.pow(Nat.0 - Nat.1) * b.pow(n - Nat.0)
    from_nat[R](Nat.0) = R.0
    mul_zero_left[R](from_nat[R](n.binom(Nat.0)))
    R.0 * from_nat[R](n.binom(Nat.0)) = R.0
    binomial_derivative_term[R](a, b, n, Nat.0) =
        R.0 * a.pow(Nat.0 - Nat.1) * b.pow(n - Nat.0)
    mul_zero_left[R](a.pow(Nat.0 - Nat.1))
    R.0 * a.pow(Nat.0 - Nat.1) = R.0
    binomial_derivative_term[R](a, b, n, Nat.0) = R.0 * b.pow(n - Nat.0)
    mul_zero_left[R](b.pow(n - Nat.0))
    R.0 * b.pow(n - Nat.0) = R.0
}

/// A positive-index derivative term is a scalar multiple of the corresponding
/// term in the preceding binomial row.
theorem binomial_derivative_term_suc[R: CommRing](
    a: R, b: R, n: Nat, k: Nat
) {
    k <= n implies
        binomial_derivative_term[R](a, b, n.suc, k.suc) =
        from_nat[R](n.suc) * binomial_term[R](a, b, n, k)
} by {
    if k <= n {
        binom_absorption_suc(n, k)
        k.suc * n.suc.binom(k.suc) = n.suc * n.binom(k)
        from_nat_mul[R](k.suc, n.suc.binom(k.suc))
        from_nat_mul[R](n.suc, n.binom(k))
        from_nat[R](k.suc) * from_nat[R](n.suc.binom(k.suc)) =
            from_nat[R](n.suc) * from_nat[R](n.binom(k))
        k.suc - Nat.1 = k
        let distance: Nat satisfy {
            k + distance = n
        }
        k.suc + distance = n.suc
        n - k = distance
        n.suc - k.suc = distance
        n.suc - k.suc = n - k
        binomial_derivative_term[R](a, b, n.suc, k.suc) =
            (from_nat[R](n.suc) * from_nat[R](n.binom(k))) *
                a.pow(k) * b.pow(n - k)
        (from_nat[R](n.suc) * from_nat[R](n.binom(k))) *
            a.pow(k) * b.pow(n - k) =
            from_nat[R](n.suc) *
                (from_nat[R](n.binom(k)) * a.pow(k) * b.pow(n - k))
        binomial_derivative_term[R](a, b, n.suc, k.suc) =
            from_nat[R](n.suc) * binomial_term[R](a, b, n, k)
    }
}

/// The sum of the formal derivative terms of `(a + b)^n` is
/// `n * (a + b)^(n - 1)`.
theorem binomial_derivative_sum[R: CommRing](a: R, b: R, n: Nat) {
    partial(binomial_derivative_term[R](a, b, n), n.suc) =
        from_nat[R](n) * (a + b).pow(n - Nat.1)
} by {
    if n = Nat.0 {
        partial(binomial_derivative_term[R](a, b, Nat.0), Nat.1) =
            binomial_derivative_term[R](a, b, Nat.0, Nat.0)
        binomial_derivative_term_zero[R](a, b, Nat.0)
        from_nat[R](Nat.0) * (a + b).pow(Nat.0 - Nat.1) = R.0
        partial(binomial_derivative_term[R](a, b, n), n.suc) =
            from_nat[R](n) * (a + b).pow(n - Nat.1)
    } else {
        let m: Nat satisfy {
            m.suc = n
        }
        m = n - Nat.1
        let scale = from_nat[R](n)

        partial_drop_first[R](binomial_derivative_term[R](a, b, n), n.suc)
        binomial_derivative_term_zero[R](a, b, n)
        partial(binomial_derivative_term[R](a, b, n), n.suc) =
            partial(compose(binomial_derivative_term[R](a, b, n), Nat.suc), n)

        forall(k: Nat) {
            if k < n {
                k <= m
                binomial_derivative_term_suc[R](a, b, m, k)
                binomial_derivative_term[R](a, b, n, k.suc) =
                    scale * binomial_term[R](a, b, m, k)
                compose(binomial_derivative_term[R](a, b, n), Nat.suc, k) =
                    mul_fn(scale, binomial_term[R](a, b, m), k)
            }
        }
        partial_pointwise_eq[R](
            compose(binomial_derivative_term[R](a, b, n), Nat.suc),
            mul_fn(scale, binomial_term[R](a, b, m)), n)
        partial(compose(binomial_derivative_term[R](a, b, n), Nat.suc), n) =
            partial(mul_fn(scale, binomial_term[R](a, b, m)), n)
        partial_scalar_mul[R](scale, binomial_term[R](a, b, m), n)
        scale * partial(binomial_term[R](a, b, m), n) =
            partial(mul_fn(scale, binomial_term[R](a, b, m)), n)
        binomial[R](a, b, m)
        partial(binomial_term[R](a, b, m), m.suc) = (a + b).pow(m)
        partial(binomial_term[R](a, b, m), n) = (a + b).pow(n - Nat.1)
        partial(binomial_derivative_term[R](a, b, n), n.suc) =
            from_nat[R](n) * (a + b).pow(n - Nat.1)
    }
}
