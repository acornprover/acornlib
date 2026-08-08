from algebra.comm_monoid import CommMonoid
from algebra.ring.ring import Ring, alternating_sign
from nat import Nat
from combinatorics import binom
from nat import from_nat
from list import partial

/// A commutative ring is a ring where multiplication is also commutative.
typeclass CommRing extends Ring, CommMonoid

/// A term in the binomial expansion of `(a + b)^n`.
/// Represents the coefficient `n.binom(k)` multiplied by `a^k b^(n-k)`.
define binomial_term[R: CommRing](a: R, b: R, n: Nat, k: Nat) -> R {
    from_nat[R](n.binom(k)) * a.pow(k) * b.pow(n - k)
}

/// The binomial theorem for commutative rings.
theorem binomial[R: CommRing](a: R, b: R, n: Nat) {
    (a + b).pow(n) = partial(binomial_term[R](a, b, n), n.suc)
}

/// The alternating sum of the binomial coefficients in a positive row is zero.
theorem alternating_binomial_row_sum[R: CommRing](n: Nat) {
    n > Nat.0 implies
        partial(binomial_term[R](-R.1, R.1, n), n.suc) = R.0
}

/// The binomial coefficient in an even position, and zero in an odd position.
define even_binomial_row_term[R: CommRing](n: Nat, k: Nat) -> R {
    if Nat.2.divides(k) {
        from_nat[R](n.binom(k))
    } else {
        R.0
    }
}

/// The binomial coefficient in an odd position, and zero in an even position.
define odd_binomial_row_term[R: CommRing](n: Nat, k: Nat) -> R {
    if Nat.2.divides(k) {
        R.0
    } else {
        from_nat[R](n.binom(k))
    }
}

/// The even- and odd-index binomial sums in a positive row are equal.
theorem even_odd_binomial_row_sums_equal[R: CommRing](n: Nat) {
    n > Nat.0 implies
        partial(even_binomial_row_term[R](n), n.suc) =
        partial(odd_binomial_row_term[R](n), n.suc)
}

/// An alternating binomial term is its coefficient multiplied by its sign.
theorem alternating_binomial_term_eq_sign_mul[R: CommRing](n: Nat, k: Nat) {
    binomial_term[R](-R.1, R.1, n, k) =
        alternating_sign[R](k) * from_nat[R](n.binom(k))
}

/// The first `m + 1` alternating terms of row `n + 1` equal the signed
/// coefficient in position `m` of row `n`.
theorem alternating_binomial_partial_row[R: CommRing](n: Nat, m: Nat) {
    partial(binomial_term[R](-R.1, R.1, n.suc), m.suc) =
        alternating_sign[R](m) * from_nat[R](n.binom(m))
}

/// The `k`th formal derivative term in the binomial expansion of `(a + b)^n`.
define binomial_derivative_term[R: CommRing](a: R, b: R, n: Nat, k: Nat) -> R {
    from_nat[R](k) * from_nat[R](n.binom(k)) *
        a.pow(k - Nat.1) * b.pow(n - k)
}

/// The sum of the formal derivative terms of `(a + b)^n` is
/// `n * (a + b)^(n - 1)`.
theorem binomial_derivative_sum[R: CommRing](a: R, b: R, n: Nat) {
    partial(binomial_derivative_term[R](a, b, n), n.suc) =
        from_nat[R](n) * (a + b).pow(n - Nat.1)
}
