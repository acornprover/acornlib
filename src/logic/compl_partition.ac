from data.basic.set import Set, set_ext, compl_contains_eq, union_contains_eq,
    intersection_contains_eq, empty_set_contains_eq, universal_set_contains_eq

/// When `u` and `v` partition the universe (disjoint and covering), `v` is the
/// complement of `u`.
theorem compl_eq_of_partition[K](u: Set[K], v: Set[K]) {
    u.intersection(v) = Set[K].empty_set and u.union(v) = Set[K].universal_set
        implies v = u.c
} by {
    if u.intersection(v) = Set[K].empty_set and u.union(v) = Set[K].universal_set {
        forall(z: K) {
            intersection_contains_eq(u, v, z)
            empty_set_contains_eq[K](z)
            union_contains_eq(u, v, z)
            universal_set_contains_eq[K](z)
            compl_contains_eq(u, z)
            if u.contains(z) {
                v.contains(z) = u.c.contains(z)
            } else {
                v.contains(z) = u.c.contains(z)
            }
            v.contains(z) = u.c.contains(z)
        }
        forall(x: K) {
            v.contains(x) = u.c.contains(x)
        }
        set_ext(v, u.c)
        v = u.c
    }
}

/// When `u` and `v` partition the universe, `u` is the complement of `v`.
theorem compl_eq_of_partition_left[K](u: Set[K], v: Set[K]) {
    u.intersection(v) = Set[K].empty_set and u.union(v) = Set[K].universal_set
        implies u = v.c
} by {
    if u.intersection(v) = Set[K].empty_set and u.union(v) = Set[K].universal_set {
        forall(z: K) {
            intersection_contains_eq(u, v, z)
            empty_set_contains_eq[K](z)
            union_contains_eq(u, v, z)
            universal_set_contains_eq[K](z)
            compl_contains_eq(v, z)
            if v.contains(z) {
                u.contains(z) = v.c.contains(z)
            } else {
                u.contains(z) = v.c.contains(z)
            }
            u.contains(z) = v.c.contains(z)
        }
        forall(x: K) {
            u.contains(x) = v.c.contains(x)
        }
        set_ext(u, v.c)
        u = v.c
    }
}

/// Disjoint sets share no element.
theorem disjoint_imp_not_both[K](u: Set[K], v: Set[K], z: K) {
    u.is_disjoint(v) implies not (u.contains(z) and v.contains(z))
} by {
    if u.is_disjoint(v) {
        forall(x: K) {
            not (u.contains(x) and v.contains(x))
        }
    }
}

/// When `u` and `v` are disjoint and cover the universe, `v` is the complement of `u`.
theorem compl_eq_of_disjoint_cover[K](u: Set[K], v: Set[K]) {
    u.is_disjoint(v) and u.union(v) = Set[K].universal_set implies v = u.c
} by {
    if u.is_disjoint(v) and u.union(v) = Set[K].universal_set {
        forall(z: K) {
            disjoint_imp_not_both(u, v, z)
            union_contains_eq(u, v, z)
            universal_set_contains_eq[K](z)
            compl_contains_eq(u, z)
            if u.contains(z) {
                v.contains(z) = u.c.contains(z)
            } else {
                v.contains(z) = u.c.contains(z)
            }
            v.contains(z) = u.c.contains(z)
        }
        forall(x: K) {
            v.contains(x) = u.c.contains(x)
        }
        set_ext(v, u.c)
        v = u.c
    }
}

/// When `u` and `v` are disjoint and cover the universe, `u` is the complement of `v`.
theorem compl_eq_of_disjoint_cover_left[K](u: Set[K], v: Set[K]) {
    u.is_disjoint(v) and u.union(v) = Set[K].universal_set implies u = v.c
} by {
    if u.is_disjoint(v) and u.union(v) = Set[K].universal_set {
        forall(z: K) {
            disjoint_imp_not_both(u, v, z)
            union_contains_eq(u, v, z)
            universal_set_contains_eq[K](z)
            compl_contains_eq(v, z)
            if v.contains(z) {
                u.contains(z) = v.c.contains(z)
            } else {
                u.contains(z) = v.c.contains(z)
            }
            u.contains(z) = v.c.contains(z)
        }
        forall(x: K) {
            u.contains(x) = v.c.contains(x)
        }
        set_ext(u, v.c)
        u = v.c
    }
}
