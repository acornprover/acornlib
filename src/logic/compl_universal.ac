from data.basic.set import Set, compl_of_compl_is_self, empty_set_compl_is_universal

/// A set whose complement is empty is the universal set.
theorem compl_empty_imp_universal[K](s: Set[K]) {
    s.c = Set[K].empty_set implies s = Set[K].universal_set
} by {
    if s.c = Set[K].empty_set {
        compl_of_compl_is_self(s)
        empty_set_compl_is_universal[K]
        s.c.c = Set[K].universal_set
    }
}
