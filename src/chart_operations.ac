from chart import Chart, chart_left_inv, chart_right_inv, chart_map_source, chart_map_target
from analysis import local_equiv_restr, local_equiv_restr_source,
    local_equiv_restr_target, local_equiv_restr_to_fun, local_equiv_restr_inv_fun,
    local_equiv_symm, local_equiv_symm_source, local_equiv_symm_target,
    local_equiv_symm_to_fun, local_equiv_symm_inv_fun, local_equiv_trans,
    local_equiv_trans_source, local_equiv_trans_target, local_equiv_trans_to_fun,
    local_equiv_trans_inv_fun
from data.basic.set import Set, set_preimage, set_preimage_contains_eq,
    intersection_contains_eq
from data.basic.functions import compose, function_extensionality

/// Restrict a chart to a subset of its source by restricting the underlying local equivalence.
define chart_restr[M, E](c: Chart[M, E], s: Set[M]) -> Chart[M, E] {
    Chart[M, E].new(local_equiv_restr[M, E](c.to_local_equiv, s))
}

/// The restricted chart has the restricted underlying local equivalence.
theorem chart_restr_to_local_equiv[M, E](c: Chart[M, E], s: Set[M]) {
    chart_restr[M, E](c, s).to_local_equiv = local_equiv_restr[M, E](c.to_local_equiv, s)
}

/// The source of a restricted chart is the restricted source of the original chart.
theorem chart_restr_source[M, E](c: Chart[M, E], s: Set[M]) {
    chart_restr[M, E](c, s).source = c.source.intersection(s)
} by {
    chart_restr_to_local_equiv(c, s)
    local_equiv_restr_source(c.to_local_equiv, s)
}

/// The target of a restricted chart is the corresponding inverse-preimage restriction.
theorem chart_restr_target[M, E](c: Chart[M, E], s: Set[M]) {
    chart_restr[M, E](c, s).target = c.target.intersection(set_preimage(c.inv_fun, s))
} by {
    chart_restr_to_local_equiv(c, s)
    local_equiv_restr_target(c.to_local_equiv, s)
}

/// The forward map of a restricted chart is unchanged.
theorem chart_restr_to_fun[M, E](c: Chart[M, E], s: Set[M]) {
    chart_restr[M, E](c, s).to_fun = c.to_fun
} by {
    chart_restr_to_local_equiv(c, s)
    local_equiv_restr_to_fun(c.to_local_equiv, s)
}

/// The inverse map of a restricted chart is unchanged.
theorem chart_restr_inv_fun[M, E](c: Chart[M, E], s: Set[M]) {
    chart_restr[M, E](c, s).inv_fun = c.inv_fun
} by {
    chart_restr_to_local_equiv(c, s)
    local_equiv_restr_inv_fun(c.to_local_equiv, s)
}

/// A point in the original source and in the restricting set lies in the restricted source.
theorem chart_restr_source_contains[M, E](c: Chart[M, E], s: Set[M], x: M) {
    c.source.contains(x) and s.contains(x) implies chart_restr[M, E](c, s).source.contains(x)
} by {
    if c.source.contains(x) and s.contains(x) {
        chart_restr_source(c, s)
        intersection_contains_eq(c.source, s, x)
        chart_restr[M, E](c, s).source.contains(x)
    }
}

/// A target point whose inverse lies in the restricting set lies in the restricted target.
theorem chart_restr_target_contains[M, E](c: Chart[M, E], s: Set[M], y: E) {
    c.target.contains(y) and s.contains(c.inv_fun(y)) implies chart_restr[M, E](c, s).target.contains(y)
} by {
    if c.target.contains(y) and s.contains(c.inv_fun(y)) {
        chart_restr_target(c, s)
        set_preimage_contains_eq(c.inv_fun, s, y)
        intersection_contains_eq(c.target, set_preimage(c.inv_fun, s), y)
        chart_restr[M, E](c, s).target.contains(y)
    }
}

/// Restricting a chart preserves the forward image membership for points that remain in the source.
theorem chart_restr_map_source[M, E](c: Chart[M, E], s: Set[M], x: M) {
    c.source.contains(x) and s.contains(x) implies chart_restr[M, E](c, s).target.contains(c.to_fun(x))
} by {
    if c.source.contains(x) and s.contains(x) {
        chart_map_source(c, x)
        chart_left_inv(c, x)
        chart_restr_target_contains(c, s, c.to_fun(x))
        chart_restr[M, E](c, s).target.contains(c.to_fun(x))
    }
}

/// Restricting a chart preserves inverse-source membership for points in the restricted target.
theorem chart_restr_map_target[M, E](c: Chart[M, E], s: Set[M], y: E) {
    c.target.contains(y) and s.contains(c.inv_fun(y)) implies
        chart_restr[M, E](c, s).source.contains(c.inv_fun(y))
} by {
    if c.target.contains(y) and s.contains(c.inv_fun(y)) {
        chart_map_target(c, y)
        chart_restr_source_contains(c, s, c.inv_fun(y))
        chart_restr[M, E](c, s).source.contains(c.inv_fun(y))
    }
}

/// The symmetric chart swaps source and target and reverses the coordinate maps.
define chart_symm[M, E](c: Chart[M, E]) -> Chart[E, M] {
    Chart[E, M].new(local_equiv_symm[M, E](c.to_local_equiv))
}

/// The symmetric chart has the symmetric underlying local equivalence.
theorem chart_symm_to_local_equiv[M, E](c: Chart[M, E]) {
    chart_symm[M, E](c).to_local_equiv = local_equiv_symm[M, E](c.to_local_equiv)
}

/// The source of the symmetric chart is the original target.
theorem chart_symm_source[M, E](c: Chart[M, E]) {
    chart_symm[M, E](c).source = c.target
} by {
    chart_symm_to_local_equiv(c)
    local_equiv_symm_source(c.to_local_equiv)
}

/// The target of the symmetric chart is the original source.
theorem chart_symm_target[M, E](c: Chart[M, E]) {
    chart_symm[M, E](c).target = c.source
} by {
    chart_symm_to_local_equiv(c)
    local_equiv_symm_target(c.to_local_equiv)
}

/// The forward map of the symmetric chart is the original inverse map.
theorem chart_symm_to_fun[M, E](c: Chart[M, E]) {
    chart_symm[M, E](c).to_fun = c.inv_fun
} by {
    chart_symm_to_local_equiv(c)
    local_equiv_symm_to_fun(c.to_local_equiv)
}

/// The inverse map of the symmetric chart is the original forward map.
theorem chart_symm_inv_fun[M, E](c: Chart[M, E]) {
    chart_symm[M, E](c).inv_fun = c.to_fun
} by {
    chart_symm_to_local_equiv(c)
    local_equiv_symm_inv_fun(c.to_local_equiv)
}

/// A point in the original target lies in the source of the symmetric chart.
theorem chart_symm_source_contains[M, E](c: Chart[M, E], y: E) {
    c.target.contains(y) implies chart_symm[M, E](c).source.contains(y)
} by {
    if c.target.contains(y) {
        chart_symm_source(c)
        chart_symm[M, E](c).source.contains(y)
    }
}

/// A point in the original source lies in the target of the symmetric chart.
theorem chart_symm_target_contains[M, E](c: Chart[M, E], x: M) {
    c.source.contains(x) implies chart_symm[M, E](c).target.contains(x)
} by {
    if c.source.contains(x) {
        chart_symm_target(c)
        chart_symm[M, E](c).target.contains(x)
    }
}

/// The symmetric chart maps original target points back into the original source.
theorem chart_symm_map_source[M, E](c: Chart[M, E], y: E) {
    c.target.contains(y) implies chart_symm[M, E](c).target.contains(c.inv_fun(y))
} by {
    if c.target.contains(y) {
        chart_map_target(c, y)
        chart_symm_target_contains(c, c.inv_fun(y))
        chart_symm[M, E](c).target.contains(c.inv_fun(y))
    }
}

/// The symmetric chart's inverse sends original source points into the original target.
theorem chart_symm_map_target[M, E](c: Chart[M, E], x: M) {
    c.source.contains(x) implies chart_symm[M, E](c).source.contains(c.to_fun(x))
} by {
    if c.source.contains(x) {
        chart_map_source(c, x)
        chart_symm_source_contains(c, c.to_fun(x))
        chart_symm[M, E](c).source.contains(c.to_fun(x))
    }
}

/// On the original target, the symmetric chart's inverse undoes its forward map.
theorem chart_symm_left_inv_apply[M, E](c: Chart[M, E], y: E) {
    c.target.contains(y) implies chart_symm[M, E](c).inv_fun(chart_symm[M, E](c).to_fun(y)) = y
} by {
    if c.target.contains(y) {
        chart_symm_to_fun(c)
        chart_symm_inv_fun(c)
        chart_right_inv(c, y)
        chart_symm[M, E](c).inv_fun(chart_symm[M, E](c).to_fun(y)) = y
    }
}

/// On the original source, the symmetric chart's forward map undoes its inverse map.
theorem chart_symm_right_inv_apply[M, E](c: Chart[M, E], x: M) {
    c.source.contains(x) implies chart_symm[M, E](c).to_fun(chart_symm[M, E](c).inv_fun(x)) = x
} by {
    if c.source.contains(x) {
        chart_symm_to_fun(c)
        chart_symm_inv_fun(c)
        chart_left_inv(c, x)
        chart_symm[M, E](c).to_fun(chart_symm[M, E](c).inv_fun(x)) = x
    }
}

/// The transition chart from `c1` coordinates to `c2` coordinates.
define chart_transition[M, E](c1: Chart[M, E], c2: Chart[M, E]) -> Chart[E, E] {
    Chart[E, E].new(local_equiv_trans[E, M, E](local_equiv_symm[M, E](c1.to_local_equiv), c2.to_local_equiv))
}

/// The transition chart has the expected underlying composition of local equivalences.
theorem chart_transition_to_local_equiv[M, E](c1: Chart[M, E], c2: Chart[M, E]) {
    chart_transition[M, E](c1, c2).to_local_equiv =
        local_equiv_trans[E, M, E](local_equiv_symm[M, E](c1.to_local_equiv), c2.to_local_equiv)
}

/// The source of the transition chart is where the first inverse lands in the second source.
theorem chart_transition_source[M, E](c1: Chart[M, E], c2: Chart[M, E]) {
    chart_transition[M, E](c1, c2).source =
        c1.target.intersection(set_preimage(c1.inv_fun, c2.source))
} by {
    let e1 = local_equiv_symm[M, E](c1.to_local_equiv)
    let e2 = c2.to_local_equiv
    chart_transition_to_local_equiv(c1, c2)
    local_equiv_trans_source(e1, e2)
    local_equiv_symm_source(c1.to_local_equiv)
    local_equiv_symm_to_fun(c1.to_local_equiv)
}

/// The target of the transition chart is where the second inverse lands in the first source.
theorem chart_transition_target[M, E](c1: Chart[M, E], c2: Chart[M, E]) {
    chart_transition[M, E](c1, c2).target =
        c2.target.intersection(set_preimage(c2.inv_fun, c1.source))
} by {
    let e1 = local_equiv_symm[M, E](c1.to_local_equiv)
    let e2 = c2.to_local_equiv
    chart_transition_to_local_equiv(c1, c2)
    local_equiv_trans_target(e1, e2)
    local_equiv_symm_target(c1.to_local_equiv)
}

/// The forward map of the transition chart is `c2.to_fun ∘ c1.inv_fun`.
theorem chart_transition_to_fun[M, E](c1: Chart[M, E], c2: Chart[M, E]) {
    chart_transition[M, E](c1, c2).to_fun = compose[E, M, E](c2.to_fun, c1.inv_fun)
} by {
    let e1 = local_equiv_symm[M, E](c1.to_local_equiv)
    let e2 = c2.to_local_equiv
    chart_transition_to_local_equiv(c1, c2)
    local_equiv_trans_to_fun(e1, e2)
    local_equiv_symm_to_fun(c1.to_local_equiv)
    forall(x: E) {
        chart_transition[M, E](c1, c2).to_fun(x) = compose[E, M, E](c2.to_fun, c1.inv_fun, x)
    }
    function_extensionality(chart_transition[M, E](c1, c2).to_fun, compose[E, M, E](c2.to_fun, c1.inv_fun))
}

/// The inverse map of the transition chart is `c1.to_fun ∘ c2.inv_fun`.
theorem chart_transition_inv_fun[M, E](c1: Chart[M, E], c2: Chart[M, E]) {
    chart_transition[M, E](c1, c2).inv_fun = compose[E, M, E](c1.to_fun, c2.inv_fun)
} by {
    let e1 = local_equiv_symm[M, E](c1.to_local_equiv)
    let e2 = c2.to_local_equiv
    chart_transition_to_local_equiv(c1, c2)
    local_equiv_trans_inv_fun(e1, e2)
    local_equiv_symm_inv_fun(c1.to_local_equiv)
    forall(y: E) {
        chart_transition[M, E](c1, c2).inv_fun(y) = compose[E, M, E](c1.to_fun, c2.inv_fun, y)
    }
    function_extensionality(chart_transition[M, E](c1, c2).inv_fun, compose[E, M, E](c1.to_fun, c2.inv_fun))
}

/// A coordinate point lies in the transition source when the first inverse lands in the second source.
theorem chart_transition_source_contains[M, E](c1: Chart[M, E], c2: Chart[M, E], y: E) {
    c1.target.contains(y) and c2.source.contains(c1.inv_fun(y)) implies
        chart_transition[M, E](c1, c2).source.contains(y)
} by {
    if c1.target.contains(y) and c2.source.contains(c1.inv_fun(y)) {
        chart_transition_source(c1, c2)
        set_preimage_contains_eq(c1.inv_fun, c2.source, y)
        intersection_contains_eq(c1.target, set_preimage(c1.inv_fun, c2.source), y)
        chart_transition[M, E](c1, c2).source.contains(y)
    }
}

/// A coordinate point lies in the transition target when the second inverse lands in the first source.
theorem chart_transition_target_contains[M, E](c1: Chart[M, E], c2: Chart[M, E], z: E) {
    c2.target.contains(z) and c1.source.contains(c2.inv_fun(z)) implies
        chart_transition[M, E](c1, c2).target.contains(z)
} by {
    if c2.target.contains(z) and c1.source.contains(c2.inv_fun(z)) {
        chart_transition_target(c1, c2)
        set_preimage_contains_eq(c2.inv_fun, c1.source, z)
        intersection_contains_eq(c2.target, set_preimage(c2.inv_fun, c1.source), z)
        chart_transition[M, E](c1, c2).target.contains(z)
    }
}

/// The transition chart maps compatible `c1` coordinates into the `c2` target.
theorem chart_transition_map_source[M, E](c1: Chart[M, E], c2: Chart[M, E], y: E) {
    c1.target.contains(y) and c2.source.contains(c1.inv_fun(y)) implies
        chart_transition[M, E](c1, c2).target.contains(c2.to_fun(c1.inv_fun(y)))
} by {
    if c1.target.contains(y) and c2.source.contains(c1.inv_fun(y)) {
        chart_map_source(c2, c1.inv_fun(y))
        chart_left_inv(c2, c1.inv_fun(y))
        chart_map_target(c1, y)
        chart_transition_target_contains(c1, c2, c2.to_fun(c1.inv_fun(y)))
        chart_transition[M, E](c1, c2).target.contains(c2.to_fun(c1.inv_fun(y)))
    }
}

/// The transition inverse maps compatible `c2` coordinates into the transition source.
theorem chart_transition_map_target[M, E](c1: Chart[M, E], c2: Chart[M, E], z: E) {
    c2.target.contains(z) and c1.source.contains(c2.inv_fun(z)) implies
        chart_transition[M, E](c1, c2).source.contains(c1.to_fun(c2.inv_fun(z)))
} by {
    if c2.target.contains(z) and c1.source.contains(c2.inv_fun(z)) {
        chart_map_source(c1, c2.inv_fun(z))
        chart_left_inv(c1, c2.inv_fun(z))
        chart_map_target(c2, z)
        chart_transition_source_contains(c1, c2, c1.to_fun(c2.inv_fun(z)))
        chart_transition[M, E](c1, c2).source.contains(c1.to_fun(c2.inv_fun(z)))
    }
}

/// On compatible source coordinates, the transition inverse undoes the transition map.
theorem chart_transition_left_inv_apply[M, E](c1: Chart[M, E], c2: Chart[M, E], y: E) {
    c1.target.contains(y) and c2.source.contains(c1.inv_fun(y)) implies
        chart_transition[M, E](c1, c2).inv_fun(c2.to_fun(c1.inv_fun(y))) = y
} by {
    if c1.target.contains(y) and c2.source.contains(c1.inv_fun(y)) {
        chart_transition_inv_fun(c1, c2)
        chart_left_inv(c2, c1.inv_fun(y))
        chart_right_inv(c1, y)
        c1.to_fun(c2.inv_fun(c2.to_fun(c1.inv_fun(y)))) = y
        chart_transition[M, E](c1, c2).inv_fun(c2.to_fun(c1.inv_fun(y))) = y
    }
}

/// On compatible target coordinates, the transition map undoes the transition inverse.
theorem chart_transition_right_inv_apply[M, E](c1: Chart[M, E], c2: Chart[M, E], z: E) {
    c2.target.contains(z) and c1.source.contains(c2.inv_fun(z)) implies
        chart_transition[M, E](c1, c2).to_fun(c1.to_fun(c2.inv_fun(z))) = z
} by {
    if c2.target.contains(z) and c1.source.contains(c2.inv_fun(z)) {
        chart_transition_to_fun(c1, c2)
        chart_left_inv(c1, c2.inv_fun(z))
        chart_right_inv(c2, z)
        c2.to_fun(c1.inv_fun(c1.to_fun(c2.inv_fun(z)))) = z
        chart_transition[M, E](c1, c2).to_fun(c1.to_fun(c2.inv_fun(z))) = z
    }
}
