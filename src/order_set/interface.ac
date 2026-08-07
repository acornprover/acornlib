/// Public interface for order theory on sets.

from data.basic.functions import compose, identity_fn
from order import PartialOrder, LinearOrder, lte_refl, lte_trans, lte_antisymm, lt_imp_lte,
    lt_trans, lt_of_lt_of_lte, lt_of_lte_of_lt, lte_max_left, lte_max_right,
    min_lte_left, min_lte_right, max_lte_of_upper_bounds, max_lt_of_upper_bounds,
    lte_min_of_bounds, lt_min_of_bounds, closed_interval, open_interval,
    left_open_interval, right_open_interval, is_monotone, is_antitone,
    is_strict_monotone, is_strict_antitone, monotone_apply, antitone_apply,
    strict_monotone_apply, strict_antitone_apply, identity_is_monotone,
    identity_is_strict_monotone
from data.basic.set import Set, set_ext, intersection_contains_eq, sets_subset_union,
    union_contains_cases, set_image, maps_into_set_image

// function_bounds.ac
/// Bounds and extrema for functions with partially ordered values.

/// The value `bound` is an upper bound for `f` on `domain`.
define is_upper_bound_on[Domain, Value: PartialOrder](
    domain: Set[Domain], f: Domain -> Value, bound: Value
) -> Bool {
    forall(point: Domain) {
        domain.contains(point) implies f(point) <= bound
    }
}

/// The value `bound` is a lower bound for `f` on `domain`.
define is_lower_bound_on[Domain, Value: PartialOrder](
    domain: Set[Domain], f: Domain -> Value, bound: Value
) -> Bool {
    forall(point: Domain) {
        domain.contains(point) implies bound <= f(point)
    }
}

/// The function `f` has an upper bound on `domain`.
define is_bounded_above_on[Domain, Value: PartialOrder](
    domain: Set[Domain], f: Domain -> Value
) -> Bool {
    exists(bound: Value) {
        is_upper_bound_on(domain, f, bound)
    }
}

/// The function `f` has a lower bound on `domain`.
define is_bounded_below_on[Domain, Value: PartialOrder](
    domain: Set[Domain], f: Domain -> Value
) -> Bool {
    exists(bound: Value) {
        is_lower_bound_on(domain, f, bound)
    }
}

/// The function `f` has both an upper and a lower bound on `domain`.
define is_bounded_on[Domain, Value: PartialOrder](
    domain: Set[Domain], f: Domain -> Value
) -> Bool {
    is_bounded_above_on(domain, f) and is_bounded_below_on(domain, f)
}

/// An upper bound dominates the value at every point of its domain.
theorem upper_bound_on_apply[Domain, Value: PartialOrder](
    domain: Set[Domain], f: Domain -> Value, bound: Value, point: Domain
) {
    is_upper_bound_on(domain, f, bound) and domain.contains(point) implies f(point) <= bound
}

/// A lower bound is dominated by the value at every point of its domain.
theorem lower_bound_on_apply[Domain, Value: PartialOrder](
    domain: Set[Domain], f: Domain -> Value, bound: Value, point: Domain
) {
    is_lower_bound_on(domain, f, bound) and domain.contains(point) implies bound <= f(point)
}

/// The function `f` attains a maximum at `point` on `domain`.
define attains_maximum_at[Domain, Value: PartialOrder](
    domain: Set[Domain], f: Domain -> Value, point: Domain
) -> Bool {
    domain.contains(point) and is_upper_bound_on(domain, f, f(point))
}

/// The function `f` attains a minimum at `point` on `domain`.
define attains_minimum_at[Domain, Value: PartialOrder](
    domain: Set[Domain], f: Domain -> Value, point: Domain
) -> Bool {
    domain.contains(point) and is_lower_bound_on(domain, f, f(point))
}

/// The function `f` attains a maximum somewhere on `domain`.
define attains_maximum_on[Domain, Value: PartialOrder](
    domain: Set[Domain], f: Domain -> Value
) -> Bool {
    exists(point: Domain) {
        attains_maximum_at(domain, f, point)
    }
}

/// The function `f` attains a minimum somewhere on `domain`.
define attains_minimum_on[Domain, Value: PartialOrder](
    domain: Set[Domain], f: Domain -> Value
) -> Bool {
    exists(point: Domain) {
        attains_minimum_at(domain, f, point)
    }
}

/// A maximum value is an upper bound.
theorem maximum_value_is_upper_bound_on[Domain, Value: PartialOrder](
    domain: Set[Domain], f: Domain -> Value, point: Domain
) {
    attains_maximum_at(domain, f, point) implies is_upper_bound_on(domain, f, f(point))
}

/// A minimum value is a lower bound.
theorem minimum_value_is_lower_bound_on[Domain, Value: PartialOrder](
    domain: Set[Domain], f: Domain -> Value, point: Domain
) {
    attains_minimum_at(domain, f, point) implies is_lower_bound_on(domain, f, f(point))
}

/// A function which attains a maximum is bounded above.
theorem attains_maximum_imp_bounded_above[Domain, Value: PartialOrder](
    domain: Set[Domain], f: Domain -> Value
) {
    attains_maximum_on(domain, f) implies is_bounded_above_on(domain, f)
}

/// A function which attains a minimum is bounded below.
theorem attains_minimum_imp_bounded_below[Domain, Value: PartialOrder](
    domain: Set[Domain], f: Domain -> Value
) {
    attains_minimum_on(domain, f) implies is_bounded_below_on(domain, f)
}

/// An upper bound on a set remains an upper bound after restricting the domain.
theorem upper_bound_on_subset[Domain, Value: PartialOrder](
    subset: Set[Domain], superset: Set[Domain], f: Domain -> Value, bound: Value
) {
    subset.subset(superset) and is_upper_bound_on(superset, f, bound)
    implies is_upper_bound_on(subset, f, bound)
}

/// A lower bound on a set remains a lower bound after restricting the domain.
theorem lower_bound_on_subset[Domain, Value: PartialOrder](
    subset: Set[Domain], superset: Set[Domain], f: Domain -> Value, bound: Value
) {
    subset.subset(superset) and is_lower_bound_on(superset, f, bound)
    implies is_lower_bound_on(subset, f, bound)
}

// function_order_on.ac
/// Pointwise order comparisons for functions restricted to a subset of their domain.

/// True if two functions are equal at every point of `domain`.
define function_eq_on[Domain, Value](
    domain: Set[Domain], left: Domain -> Value, right: Domain -> Value
) -> Bool {
    forall(point: Domain) {
        domain.contains(point) implies left(point) = right(point)
    }
}

/// True if `lower` lies pointwise below `upper` on `domain`.
define function_lte_on[Domain, Value: PartialOrder](
    domain: Set[Domain], lower: Domain -> Value, upper: Domain -> Value
) -> Bool {
    forall(point: Domain) {
        domain.contains(point) implies lower(point) <= upper(point)
    }
}

/// True if `lower` lies pointwise strictly below `upper` on `domain`.
define function_lt_on[Domain, Value: PartialOrder](
    domain: Set[Domain], lower: Domain -> Value, upper: Domain -> Value
) -> Bool {
    forall(point: Domain) {
        domain.contains(point) implies lower(point) < upper(point)
    }
}

/// True if `middle` lies pointwise between `lower` and `upper` on `domain`.
define function_between_on[Domain, Value: PartialOrder](
    domain: Set[Domain], lower: Domain -> Value, middle: Domain -> Value, upper: Domain -> Value
) -> Bool {
    function_lte_on(domain, lower, middle) and function_lte_on(domain, middle, upper)
}

/// True if `middle` lies pointwise strictly between `lower` and `upper` on `domain`.
define function_strict_between_on[Domain, Value: PartialOrder](
    domain: Set[Domain], lower: Domain -> Value, middle: Domain -> Value, upper: Domain -> Value
) -> Bool {
    function_lt_on(domain, lower, middle) and function_lt_on(domain, middle, upper)
}

/// Pointwise equality on a set gives equality at any point of the set.
theorem function_eq_on_apply[Domain, Value](
    domain: Set[Domain], left: Domain -> Value, right: Domain -> Value, point: Domain
) {
    function_eq_on(domain, left, right) and domain.contains(point)
    implies left(point) = right(point)
}

/// Pointwise equality on a set is symmetric.
theorem function_eq_on_symm[Domain, Value](
    domain: Set[Domain], left: Domain -> Value, right: Domain -> Value
) {
    function_eq_on(domain, left, right) implies function_eq_on(domain, right, left)
}

/// Pointwise equality on a set is transitive.
theorem function_eq_on_trans[Domain, Value](
    domain: Set[Domain], f: Domain -> Value, g: Domain -> Value, h: Domain -> Value
) {
    function_eq_on(domain, f, g) and function_eq_on(domain, g, h) implies function_eq_on(domain, f, h)
}

/// Pointwise equality remains true after restricting the domain.
theorem function_eq_on_subset[Domain, Value](
    subset: Set[Domain], superset: Set[Domain], left: Domain -> Value, right: Domain -> Value
) {
    subset.subset(superset) and function_eq_on(superset, left, right)
    implies function_eq_on(subset, left, right)
}

/// A pointwise comparison on a set gives the comparison at any point of the set.
theorem function_lte_on_apply[Domain, Value: PartialOrder](
    domain: Set[Domain], lower: Domain -> Value, upper: Domain -> Value, point: Domain
) {
    function_lte_on(domain, lower, upper) and domain.contains(point)
    implies lower(point) <= upper(point)
}

/// A strict pointwise comparison on a set gives the strict comparison at any point of the set.
theorem function_lt_on_apply[Domain, Value: PartialOrder](
    domain: Set[Domain], lower: Domain -> Value, upper: Domain -> Value, point: Domain
) {
    function_lt_on(domain, lower, upper) and domain.contains(point)
    implies lower(point) < upper(point)
}

/// The pointwise order on a set is reflexive.
theorem function_lte_on_refl[Domain, Value: PartialOrder](
    domain: Set[Domain], f: Domain -> Value
) {
    function_lte_on(domain, f, f)
}

/// Pointwise equality on a set implies pointwise order on that set.
theorem function_lte_on_of_eq_on[Domain, Value: PartialOrder](
    domain: Set[Domain], left: Domain -> Value, right: Domain -> Value
) {
    function_eq_on(domain, left, right) implies function_lte_on(domain, left, right)
}

/// Pointwise order on a set is transitive.
theorem function_lte_on_trans[Domain, Value: PartialOrder](
    domain: Set[Domain], f: Domain -> Value, g: Domain -> Value, h: Domain -> Value
) {
    function_lte_on(domain, f, g) and function_lte_on(domain, g, h)
    implies function_lte_on(domain, f, h)
}

/// Mutually pointwise ordered functions are pointwise equal on the set.
theorem function_eq_on_of_lte_on_both[Domain, Value: PartialOrder](
    domain: Set[Domain], left: Domain -> Value, right: Domain -> Value
) {
    function_lte_on(domain, left, right) and function_lte_on(domain, right, left)
    implies function_eq_on(domain, left, right)
}

/// Replacing the lower function by an equal one preserves pointwise order on the set.
theorem function_lte_on_congr_left[Domain, Value: PartialOrder](
    domain: Set[Domain], f: Domain -> Value, g: Domain -> Value, h: Domain -> Value
) {
    function_eq_on(domain, f, g) and function_lte_on(domain, g, h)
    implies function_lte_on(domain, f, h)
}

/// Replacing the upper function by an equal one preserves pointwise order on the set.
theorem function_lte_on_congr_right[Domain, Value: PartialOrder](
    domain: Set[Domain], f: Domain -> Value, g: Domain -> Value, h: Domain -> Value
) {
    function_lte_on(domain, f, g) and function_eq_on(domain, g, h)
    implies function_lte_on(domain, f, h)
}

/// Replacing the lower function by an equal one reflects pointwise order on the set.
theorem function_lte_on_reflect_congr_left[Domain, Value: PartialOrder](
    domain: Set[Domain], f: Domain -> Value, g: Domain -> Value, h: Domain -> Value
) {
    function_eq_on(domain, f, g) and function_lte_on(domain, f, h)
    implies function_lte_on(domain, g, h)
}

/// Replacing the upper function by an equal one reflects pointwise order on the set.
theorem function_lte_on_reflect_congr_right[Domain, Value: PartialOrder](
    domain: Set[Domain], f: Domain -> Value, g: Domain -> Value, h: Domain -> Value
) {
    function_lte_on(domain, f, h) and function_eq_on(domain, g, h)
    implies function_lte_on(domain, f, g)
}

/// Replacing both sides by equal functions preserves pointwise order on the set.
theorem function_lte_on_congr[Domain, Value: PartialOrder](
    domain: Set[Domain], f: Domain -> Value, g: Domain -> Value,
    h: Domain -> Value, k: Domain -> Value
) {
    function_eq_on(domain, f, g) and function_lte_on(domain, g, h) and function_eq_on(domain, h, k)
    implies function_lte_on(domain, f, k)
}

/// Equal functions are pointwise ordered in both directions on the set.
theorem function_lte_on_pair_of_eq_on[Domain, Value: PartialOrder](
    domain: Set[Domain], left: Domain -> Value, right: Domain -> Value
) {
    function_eq_on(domain, left, right)
    implies function_lte_on(domain, left, right) and function_lte_on(domain, right, left)
}

/// Pointwise strict order on a set is transitive.
theorem function_lt_on_trans[Domain, Value: PartialOrder](
    domain: Set[Domain], f: Domain -> Value, g: Domain -> Value, h: Domain -> Value
) {
    function_lt_on(domain, f, g) and function_lt_on(domain, g, h)
    implies function_lt_on(domain, f, h)
}

/// A strict comparison followed by a non-strict comparison gives a strict comparison.
theorem function_lt_lte_on_trans[Domain, Value: PartialOrder](
    domain: Set[Domain], f: Domain -> Value, g: Domain -> Value, h: Domain -> Value
) {
    function_lt_on(domain, f, g) and function_lte_on(domain, g, h)
    implies function_lt_on(domain, f, h)
}

/// A non-strict comparison followed by a strict comparison gives a strict comparison.
theorem function_lte_lt_on_trans[Domain, Value: PartialOrder](
    domain: Set[Domain], f: Domain -> Value, g: Domain -> Value, h: Domain -> Value
) {
    function_lte_on(domain, f, g) and function_lt_on(domain, g, h)
    implies function_lt_on(domain, f, h)
}

/// Pointwise strict order on a set implies pointwise order on the same set.
theorem function_lt_on_imp_lte_on[Domain, Value: PartialOrder](
    domain: Set[Domain], lower: Domain -> Value, upper: Domain -> Value
) {
    function_lt_on(domain, lower, upper) implies function_lte_on(domain, lower, upper)
}

/// Replacing the lower function by an equal one preserves strict pointwise order on the set.
theorem function_lt_on_congr_left[Domain, Value: PartialOrder](
    domain: Set[Domain], f: Domain -> Value, g: Domain -> Value, h: Domain -> Value
) {
    function_eq_on(domain, f, g) and function_lt_on(domain, g, h)
    implies function_lt_on(domain, f, h)
}

/// Replacing the upper function by an equal one preserves strict pointwise order on the set.
theorem function_lt_on_congr_right[Domain, Value: PartialOrder](
    domain: Set[Domain], f: Domain -> Value, g: Domain -> Value, h: Domain -> Value
) {
    function_lt_on(domain, f, g) and function_eq_on(domain, g, h)
    implies function_lt_on(domain, f, h)
}

/// Replacing the lower function by an equal one reflects strict pointwise order on the set.
theorem function_lt_on_reflect_congr_left[Domain, Value: PartialOrder](
    domain: Set[Domain], f: Domain -> Value, g: Domain -> Value, h: Domain -> Value
) {
    function_eq_on(domain, f, g) and function_lt_on(domain, f, h)
    implies function_lt_on(domain, g, h)
}

/// Replacing the upper function by an equal one reflects strict pointwise order on the set.
theorem function_lt_on_reflect_congr_right[Domain, Value: PartialOrder](
    domain: Set[Domain], f: Domain -> Value, g: Domain -> Value, h: Domain -> Value
) {
    function_lt_on(domain, f, h) and function_eq_on(domain, g, h)
    implies function_lt_on(domain, f, g)
}

/// Replacing both sides by equal functions preserves strict pointwise order on the set.
theorem function_lt_on_congr[Domain, Value: PartialOrder](
    domain: Set[Domain], f: Domain -> Value, g: Domain -> Value,
    h: Domain -> Value, k: Domain -> Value
) {
    function_eq_on(domain, f, g) and function_lt_on(domain, g, h) and function_eq_on(domain, h, k)
    implies function_lt_on(domain, f, k)
}

/// A pointwise between relation gives the lower comparison.
theorem function_between_on_left[Domain, Value: PartialOrder](
    domain: Set[Domain], lower: Domain -> Value, middle: Domain -> Value, upper: Domain -> Value
) {
    function_between_on(domain, lower, middle, upper) implies function_lte_on(domain, lower, middle)
}

/// A pointwise between relation gives the upper comparison.
theorem function_between_on_right[Domain, Value: PartialOrder](
    domain: Set[Domain], lower: Domain -> Value, middle: Domain -> Value, upper: Domain -> Value
) {
    function_between_on(domain, lower, middle, upper) implies function_lte_on(domain, middle, upper)
}

/// A strict pointwise between relation gives the lower strict comparison.
theorem function_strict_between_on_left[Domain, Value: PartialOrder](
    domain: Set[Domain], lower: Domain -> Value, middle: Domain -> Value, upper: Domain -> Value
) {
    function_strict_between_on(domain, lower, middle, upper) implies function_lt_on(domain, lower, middle)
}

/// A strict pointwise between relation gives the upper strict comparison.
theorem function_strict_between_on_right[Domain, Value: PartialOrder](
    domain: Set[Domain], lower: Domain -> Value, middle: Domain -> Value, upper: Domain -> Value
) {
    function_strict_between_on(domain, lower, middle, upper) implies function_lt_on(domain, middle, upper)
}

/// A strict pointwise between relation implies a non-strict pointwise between relation.
theorem function_between_on_of_strict_between_on[Domain, Value: PartialOrder](
    domain: Set[Domain], lower: Domain -> Value, middle: Domain -> Value, upper: Domain -> Value
) {
    function_strict_between_on(domain, lower, middle, upper)
    implies function_between_on(domain, lower, middle, upper)
}

/// A pointwise between relation remains true after restricting the domain.
theorem function_between_on_subset[Domain, Value: PartialOrder](
    subset: Set[Domain], superset: Set[Domain], lower: Domain -> Value,
    middle: Domain -> Value, upper: Domain -> Value
) {
    subset.subset(superset) and function_between_on(superset, lower, middle, upper)
    implies function_between_on(subset, lower, middle, upper)
}

/// A strict pointwise between relation remains true after restricting the domain.
theorem function_strict_between_on_subset[Domain, Value: PartialOrder](
    subset: Set[Domain], superset: Set[Domain], lower: Domain -> Value,
    middle: Domain -> Value, upper: Domain -> Value
) {
    subset.subset(superset) and function_strict_between_on(superset, lower, middle, upper)
    implies function_strict_between_on(subset, lower, middle, upper)
}

/// A pointwise order comparison remains true after restricting the domain.
theorem function_lte_on_subset[Domain, Value: PartialOrder](
    subset: Set[Domain], superset: Set[Domain], lower: Domain -> Value, upper: Domain -> Value
) {
    subset.subset(superset) and function_lte_on(superset, lower, upper)
    implies function_lte_on(subset, lower, upper)
}

/// A strict pointwise comparison remains true after restricting the domain.
theorem function_lt_on_subset[Domain, Value: PartialOrder](
    subset: Set[Domain], superset: Set[Domain], lower: Domain -> Value, upper: Domain -> Value
) {
    subset.subset(superset) and function_lt_on(superset, lower, upper)
    implies function_lt_on(subset, lower, upper)
}

/// Pointwise equality on two domains gives pointwise equality on their union.
theorem function_eq_on_union[Domain, Value](
    left: Set[Domain], right: Set[Domain], f: Domain -> Value, g: Domain -> Value
) {
    function_eq_on(left, f, g) and function_eq_on(right, f, g)
    implies function_eq_on(left.union(right), f, g)
}

/// Pointwise equality on a union restricts to the left domain.
theorem function_eq_on_union_left[Domain, Value](
    left: Set[Domain], right: Set[Domain], f: Domain -> Value, g: Domain -> Value
) {
    function_eq_on(left.union(right), f, g) implies function_eq_on(left, f, g)
}

/// Pointwise equality on a union restricts to the right domain.
theorem function_eq_on_union_right[Domain, Value](
    left: Set[Domain], right: Set[Domain], f: Domain -> Value, g: Domain -> Value
) {
    function_eq_on(left.union(right), f, g) implies function_eq_on(right, f, g)
}

/// Pointwise order on two domains gives pointwise order on their union.
theorem function_lte_on_union[Domain, Value: PartialOrder](
    left: Set[Domain], right: Set[Domain], lower: Domain -> Value, upper: Domain -> Value
) {
    function_lte_on(left, lower, upper) and function_lte_on(right, lower, upper)
    implies function_lte_on(left.union(right), lower, upper)
}

/// Pointwise order on a union restricts to the left domain.
theorem function_lte_on_union_left[Domain, Value: PartialOrder](
    left: Set[Domain], right: Set[Domain], lower: Domain -> Value, upper: Domain -> Value
) {
    function_lte_on(left.union(right), lower, upper) implies function_lte_on(left, lower, upper)
}

/// Pointwise order on a union restricts to the right domain.
theorem function_lte_on_union_right[Domain, Value: PartialOrder](
    left: Set[Domain], right: Set[Domain], lower: Domain -> Value, upper: Domain -> Value
) {
    function_lte_on(left.union(right), lower, upper) implies function_lte_on(right, lower, upper)
}

/// Strict pointwise order on two domains gives strict pointwise order on their union.
theorem function_lt_on_union[Domain, Value: PartialOrder](
    left: Set[Domain], right: Set[Domain], lower: Domain -> Value, upper: Domain -> Value
) {
    function_lt_on(left, lower, upper) and function_lt_on(right, lower, upper)
    implies function_lt_on(left.union(right), lower, upper)
}

/// Strict pointwise order on a union restricts to the left domain.
theorem function_lt_on_union_left[Domain, Value: PartialOrder](
    left: Set[Domain], right: Set[Domain], lower: Domain -> Value, upper: Domain -> Value
) {
    function_lt_on(left.union(right), lower, upper) implies function_lt_on(left, lower, upper)
}

/// Strict pointwise order on a union restricts to the right domain.
theorem function_lt_on_union_right[Domain, Value: PartialOrder](
    left: Set[Domain], right: Set[Domain], lower: Domain -> Value, upper: Domain -> Value
) {
    function_lt_on(left.union(right), lower, upper) implies function_lt_on(right, lower, upper)
}

/// Pointwise betweenness on two domains gives pointwise betweenness on their union.
theorem function_between_on_union[Domain, Value: PartialOrder](
    left: Set[Domain], right: Set[Domain], lower: Domain -> Value,
    middle: Domain -> Value, upper: Domain -> Value
) {
    function_between_on(left, lower, middle, upper) and function_between_on(right, lower, middle, upper)
    implies function_between_on(left.union(right), lower, middle, upper)
}

/// Pointwise betweenness on a union restricts to the left domain.
theorem function_between_on_union_left[Domain, Value: PartialOrder](
    left: Set[Domain], right: Set[Domain], lower: Domain -> Value,
    middle: Domain -> Value, upper: Domain -> Value
) {
    function_between_on(left.union(right), lower, middle, upper)
    implies function_between_on(left, lower, middle, upper)
}

/// Pointwise betweenness on a union restricts to the right domain.
theorem function_between_on_union_right[Domain, Value: PartialOrder](
    left: Set[Domain], right: Set[Domain], lower: Domain -> Value,
    middle: Domain -> Value, upper: Domain -> Value
) {
    function_between_on(left.union(right), lower, middle, upper)
    implies function_between_on(right, lower, middle, upper)
}

/// Strict pointwise betweenness on two domains gives strict pointwise betweenness on their union.
theorem function_strict_between_on_union[Domain, Value: PartialOrder](
    left: Set[Domain], right: Set[Domain], lower: Domain -> Value,
    middle: Domain -> Value, upper: Domain -> Value
) {
    function_strict_between_on(left, lower, middle, upper)
    and function_strict_between_on(right, lower, middle, upper)
    implies function_strict_between_on(left.union(right), lower, middle, upper)
}

/// Strict pointwise betweenness on a union restricts to the left domain.
theorem function_strict_between_on_union_left[Domain, Value: PartialOrder](
    left: Set[Domain], right: Set[Domain], lower: Domain -> Value,
    middle: Domain -> Value, upper: Domain -> Value
) {
    function_strict_between_on(left.union(right), lower, middle, upper)
    implies function_strict_between_on(left, lower, middle, upper)
}

/// Strict pointwise betweenness on a union restricts to the right domain.
theorem function_strict_between_on_union_right[Domain, Value: PartialOrder](
    left: Set[Domain], right: Set[Domain], lower: Domain -> Value,
    middle: Domain -> Value, upper: Domain -> Value
) {
    function_strict_between_on(left.union(right), lower, middle, upper)
    implies function_strict_between_on(right, lower, middle, upper)
}

// function_order_bounds.ac
/// Bounds transported along pointwise function comparisons on a set.

/// An upper bound for a larger function is an upper bound for any pointwise smaller function.
theorem upper_bound_on_of_function_lte_on[Domain, Value: PartialOrder](
    domain: Set[Domain], lower: Domain -> Value, upper: Domain -> Value, bound: Value
) {
    function_lte_on(domain, lower, upper) and is_upper_bound_on(domain, upper, bound)
    implies is_upper_bound_on(domain, lower, bound)
}

/// A lower bound for a smaller function is a lower bound for any pointwise larger function.
theorem lower_bound_on_of_function_lte_on[Domain, Value: PartialOrder](
    domain: Set[Domain], lower: Domain -> Value, upper: Domain -> Value, bound: Value
) {
    function_lte_on(domain, lower, upper) and is_lower_bound_on(domain, lower, bound)
    implies is_lower_bound_on(domain, upper, bound)
}

/// Boundedness above descends along a pointwise comparison.
theorem bounded_above_on_of_function_lte_on[Domain, Value: PartialOrder](
    domain: Set[Domain], lower: Domain -> Value, upper: Domain -> Value
) {
    function_lte_on(domain, lower, upper) and is_bounded_above_on(domain, upper)
    implies is_bounded_above_on(domain, lower)
}

/// Boundedness below ascends along a pointwise comparison.
theorem bounded_below_on_of_function_lte_on[Domain, Value: PartialOrder](
    domain: Set[Domain], lower: Domain -> Value, upper: Domain -> Value
) {
    function_lte_on(domain, lower, upper) and is_bounded_below_on(domain, lower)
    implies is_bounded_below_on(domain, upper)
}

/// An upper bound descends along a strict pointwise comparison.
theorem upper_bound_on_of_function_lt_on[Domain, Value: PartialOrder](
    domain: Set[Domain], lower: Domain -> Value, upper: Domain -> Value, bound: Value
) {
    function_lt_on(domain, lower, upper) and is_upper_bound_on(domain, upper, bound)
    implies is_upper_bound_on(domain, lower, bound)
}

/// A lower bound ascends along a strict pointwise comparison.
theorem lower_bound_on_of_function_lt_on[Domain, Value: PartialOrder](
    domain: Set[Domain], lower: Domain -> Value, upper: Domain -> Value, bound: Value
) {
    function_lt_on(domain, lower, upper) and is_lower_bound_on(domain, lower, bound)
    implies is_lower_bound_on(domain, upper, bound)
}

/// Boundedness above descends along a strict pointwise comparison.
theorem bounded_above_on_of_function_lt_on[Domain, Value: PartialOrder](
    domain: Set[Domain], lower: Domain -> Value, upper: Domain -> Value
) {
    function_lt_on(domain, lower, upper) and is_bounded_above_on(domain, upper)
    implies is_bounded_above_on(domain, lower)
}

/// Boundedness below ascends along a strict pointwise comparison.
theorem bounded_below_on_of_function_lt_on[Domain, Value: PartialOrder](
    domain: Set[Domain], lower: Domain -> Value, upper: Domain -> Value
) {
    function_lt_on(domain, lower, upper) and is_bounded_below_on(domain, lower)
    implies is_bounded_below_on(domain, upper)
}

/// Both sides of a pointwise comparison are bounded above when the larger side is bounded above.
theorem bounded_above_on_pair_of_function_lte_on[Domain, Value: PartialOrder](
    domain: Set[Domain], lower: Domain -> Value, upper: Domain -> Value
) {
    function_lte_on(domain, lower, upper) and is_bounded_above_on(domain, upper)
    implies is_bounded_above_on(domain, lower) and is_bounded_above_on(domain, upper)
}

/// Both sides of a pointwise comparison are bounded below when the smaller side is bounded below.
theorem bounded_below_on_pair_of_function_lte_on[Domain, Value: PartialOrder](
    domain: Set[Domain], lower: Domain -> Value, upper: Domain -> Value
) {
    function_lte_on(domain, lower, upper) and is_bounded_below_on(domain, lower)
    implies is_bounded_below_on(domain, lower) and is_bounded_below_on(domain, upper)
}

/// The middle of a pointwise sandwich is bounded above when the upper function is bounded above.
theorem bounded_above_on_middle_of_function_lte_on[Domain, Value: PartialOrder](
    domain: Set[Domain], lower: Domain -> Value, middle: Domain -> Value, upper: Domain -> Value
) {
    function_lte_on(domain, lower, middle) and function_lte_on(domain, middle, upper) and
    is_bounded_above_on(domain, upper)
    implies is_bounded_above_on(domain, middle)
}

/// The middle of a pointwise sandwich is bounded below when the lower function is bounded below.
theorem bounded_below_on_middle_of_function_lte_on[Domain, Value: PartialOrder](
    domain: Set[Domain], lower: Domain -> Value, middle: Domain -> Value, upper: Domain -> Value
) {
    function_lte_on(domain, lower, middle) and function_lte_on(domain, middle, upper) and
    is_bounded_below_on(domain, lower)
    implies is_bounded_below_on(domain, middle)
}

/// Both sides of a strict pointwise comparison are bounded above when the larger side is bounded above.
theorem bounded_above_on_pair_of_function_lt_on[Domain, Value: PartialOrder](
    domain: Set[Domain], lower: Domain -> Value, upper: Domain -> Value
) {
    function_lt_on(domain, lower, upper) and is_bounded_above_on(domain, upper)
    implies is_bounded_above_on(domain, lower) and is_bounded_above_on(domain, upper)
}

/// Both sides of a strict pointwise comparison are bounded below when the smaller side is bounded below.
theorem bounded_below_on_pair_of_function_lt_on[Domain, Value: PartialOrder](
    domain: Set[Domain], lower: Domain -> Value, upper: Domain -> Value
) {
    function_lt_on(domain, lower, upper) and is_bounded_below_on(domain, lower)
    implies is_bounded_below_on(domain, lower) and is_bounded_below_on(domain, upper)
}

/// The middle of a strict pointwise sandwich is bounded above when the upper function is bounded above.
theorem bounded_above_on_middle_of_function_lt_on[Domain, Value: PartialOrder](
    domain: Set[Domain], lower: Domain -> Value, middle: Domain -> Value, upper: Domain -> Value
) {
    function_lt_on(domain, lower, middle) and function_lt_on(domain, middle, upper) and
    is_bounded_above_on(domain, upper)
    implies is_bounded_above_on(domain, middle)
}

/// The middle of a strict pointwise sandwich is bounded below when the lower function is bounded below.
theorem bounded_below_on_middle_of_function_lt_on[Domain, Value: PartialOrder](
    domain: Set[Domain], lower: Domain -> Value, middle: Domain -> Value, upper: Domain -> Value
) {
    function_lt_on(domain, lower, middle) and function_lt_on(domain, middle, upper) and
    is_bounded_below_on(domain, lower)
    implies is_bounded_below_on(domain, middle)
}

/// A pointwise smaller function is bounded when it has a lower bound and the larger function has an upper bound.
theorem bounded_on_lower_of_function_lte_on[Domain, Value: PartialOrder](
    domain: Set[Domain], lower: Domain -> Value, upper: Domain -> Value
) {
    function_lte_on(domain, lower, upper) and is_bounded_below_on(domain, lower) and
    is_bounded_above_on(domain, upper)
    implies is_bounded_on(domain, lower)
}

/// A pointwise larger function is bounded when the smaller function has a lower bound and it has an upper bound.
theorem bounded_on_upper_of_function_lte_on[Domain, Value: PartialOrder](
    domain: Set[Domain], lower: Domain -> Value, upper: Domain -> Value
) {
    function_lte_on(domain, lower, upper) and is_bounded_below_on(domain, lower) and
    is_bounded_above_on(domain, upper)
    implies is_bounded_on(domain, upper)
}

/// Both sides of a pointwise comparison are bounded when the lower side has a lower bound and the upper side has an upper bound.
theorem bounded_on_pair_of_function_lte_on[Domain, Value: PartialOrder](
    domain: Set[Domain], lower: Domain -> Value, upper: Domain -> Value
) {
    function_lte_on(domain, lower, upper) and is_bounded_below_on(domain, lower) and
    is_bounded_above_on(domain, upper)
    implies is_bounded_on(domain, lower) and is_bounded_on(domain, upper)
}

/// A function between lower and upper pointwise bounds is bounded on the same set.
theorem bounded_on_middle_of_function_lte_on[Domain, Value: PartialOrder](
    domain: Set[Domain], lower: Domain -> Value, middle: Domain -> Value, upper: Domain -> Value
) {
    function_lte_on(domain, lower, middle) and function_lte_on(domain, middle, upper) and
    is_bounded_below_on(domain, lower) and is_bounded_above_on(domain, upper)
    implies is_bounded_on(domain, middle)
}

/// A strictly smaller function is bounded under the corresponding endpoint bounds.
theorem bounded_on_lower_of_function_lt_on[Domain, Value: PartialOrder](
    domain: Set[Domain], lower: Domain -> Value, upper: Domain -> Value
) {
    function_lt_on(domain, lower, upper) and is_bounded_below_on(domain, lower) and
    is_bounded_above_on(domain, upper)
    implies is_bounded_on(domain, lower)
}

/// A strictly larger function is bounded under the corresponding endpoint bounds.
theorem bounded_on_upper_of_function_lt_on[Domain, Value: PartialOrder](
    domain: Set[Domain], lower: Domain -> Value, upper: Domain -> Value
) {
    function_lt_on(domain, lower, upper) and is_bounded_below_on(domain, lower) and
    is_bounded_above_on(domain, upper)
    implies is_bounded_on(domain, upper)
}

/// Both sides of a strict pointwise comparison are bounded under the corresponding endpoint bounds.
theorem bounded_on_pair_of_function_lt_on[Domain, Value: PartialOrder](
    domain: Set[Domain], lower: Domain -> Value, upper: Domain -> Value
) {
    function_lt_on(domain, lower, upper) and is_bounded_below_on(domain, lower) and
    is_bounded_above_on(domain, upper)
    implies is_bounded_on(domain, lower) and is_bounded_on(domain, upper)
}

/// A function between strict pointwise lower and upper bounds is bounded on the same set.
theorem bounded_on_middle_of_function_lt_on[Domain, Value: PartialOrder](
    domain: Set[Domain], lower: Domain -> Value, middle: Domain -> Value, upper: Domain -> Value
) {
    function_lt_on(domain, lower, middle) and function_lt_on(domain, middle, upper) and
    is_bounded_below_on(domain, lower) and is_bounded_above_on(domain, upper)
    implies is_bounded_on(domain, middle)
}


/// An upper bound for the upper function bounds the middle function in a pointwise between relation.
theorem upper_bound_on_middle_of_function_between_on[Domain, Value: PartialOrder](
    domain: Set[Domain], lower: Domain -> Value, middle: Domain -> Value,
    upper: Domain -> Value, bound: Value
) {
    function_between_on(domain, lower, middle, upper) and is_upper_bound_on(domain, upper, bound)
    implies is_upper_bound_on(domain, middle, bound)
}

/// A lower bound for the lower function bounds the middle function in a pointwise between relation.
theorem lower_bound_on_middle_of_function_between_on[Domain, Value: PartialOrder](
    domain: Set[Domain], lower: Domain -> Value, middle: Domain -> Value,
    upper: Domain -> Value, bound: Value
) {
    function_between_on(domain, lower, middle, upper) and is_lower_bound_on(domain, lower, bound)
    implies is_lower_bound_on(domain, middle, bound)
}

/// An upper bound for the upper function bounds the lower function in a pointwise between relation.
theorem upper_bound_on_lower_of_function_between_on[Domain, Value: PartialOrder](
    domain: Set[Domain], lower: Domain -> Value, middle: Domain -> Value,
    upper: Domain -> Value, bound: Value
) {
    function_between_on(domain, lower, middle, upper) and is_upper_bound_on(domain, upper, bound)
    implies is_upper_bound_on(domain, lower, bound)
}

/// A lower bound for the lower function bounds the upper function in a pointwise between relation.
theorem lower_bound_on_upper_of_function_between_on[Domain, Value: PartialOrder](
    domain: Set[Domain], lower: Domain -> Value, middle: Domain -> Value,
    upper: Domain -> Value, bound: Value
) {
    function_between_on(domain, lower, middle, upper) and is_lower_bound_on(domain, lower, bound)
    implies is_lower_bound_on(domain, upper, bound)
}

/// An upper bound for the upper function bounds the middle function in a strict pointwise between relation.
theorem upper_bound_on_middle_of_function_strict_between_on[Domain, Value: PartialOrder](
    domain: Set[Domain], lower: Domain -> Value, middle: Domain -> Value,
    upper: Domain -> Value, bound: Value
) {
    function_strict_between_on(domain, lower, middle, upper) and is_upper_bound_on(domain, upper, bound)
    implies is_upper_bound_on(domain, middle, bound)
}

/// A lower bound for the lower function bounds the middle function in a strict pointwise between relation.
theorem lower_bound_on_middle_of_function_strict_between_on[Domain, Value: PartialOrder](
    domain: Set[Domain], lower: Domain -> Value, middle: Domain -> Value,
    upper: Domain -> Value, bound: Value
) {
    function_strict_between_on(domain, lower, middle, upper) and is_lower_bound_on(domain, lower, bound)
    implies is_lower_bound_on(domain, middle, bound)
}

/// An upper bound for the upper function bounds the lower function in a strict pointwise between relation.
theorem upper_bound_on_lower_of_function_strict_between_on[Domain, Value: PartialOrder](
    domain: Set[Domain], lower: Domain -> Value, middle: Domain -> Value,
    upper: Domain -> Value, bound: Value
) {
    function_strict_between_on(domain, lower, middle, upper) and is_upper_bound_on(domain, upper, bound)
    implies is_upper_bound_on(domain, lower, bound)
}

/// A lower bound for the lower function bounds the upper function in a strict pointwise between relation.
theorem lower_bound_on_upper_of_function_strict_between_on[Domain, Value: PartialOrder](
    domain: Set[Domain], lower: Domain -> Value, middle: Domain -> Value,
    upper: Domain -> Value, bound: Value
) {
    function_strict_between_on(domain, lower, middle, upper) and is_lower_bound_on(domain, lower, bound)
    implies is_lower_bound_on(domain, upper, bound)
}

/// The middle of a pointwise between relation is bounded above when the upper function is bounded above.
theorem bounded_above_on_middle_of_function_between_on[Domain, Value: PartialOrder](
    domain: Set[Domain], lower: Domain -> Value, middle: Domain -> Value, upper: Domain -> Value
) {
    function_between_on(domain, lower, middle, upper) and is_bounded_above_on(domain, upper)
    implies is_bounded_above_on(domain, middle)
}

/// The middle of a pointwise between relation is bounded below when the lower function is bounded below.
theorem bounded_below_on_middle_of_function_between_on[Domain, Value: PartialOrder](
    domain: Set[Domain], lower: Domain -> Value, middle: Domain -> Value, upper: Domain -> Value
) {
    function_between_on(domain, lower, middle, upper) and is_bounded_below_on(domain, lower)
    implies is_bounded_below_on(domain, middle)
}

/// The lower function in a pointwise between relation is bounded above when the upper function is bounded above.
theorem bounded_above_on_lower_of_function_between_on[Domain, Value: PartialOrder](
    domain: Set[Domain], lower: Domain -> Value, middle: Domain -> Value, upper: Domain -> Value
) {
    function_between_on(domain, lower, middle, upper) and is_bounded_above_on(domain, upper)
    implies is_bounded_above_on(domain, lower)
}

/// The upper function in a pointwise between relation is bounded below when the lower function is bounded below.
theorem bounded_below_on_upper_of_function_between_on[Domain, Value: PartialOrder](
    domain: Set[Domain], lower: Domain -> Value, middle: Domain -> Value, upper: Domain -> Value
) {
    function_between_on(domain, lower, middle, upper) and is_bounded_below_on(domain, lower)
    implies is_bounded_below_on(domain, upper)
}

/// The middle of a pointwise between relation is bounded under endpoint bounds.
theorem bounded_on_middle_of_function_between_on[Domain, Value: PartialOrder](
    domain: Set[Domain], lower: Domain -> Value, middle: Domain -> Value, upper: Domain -> Value
) {
    function_between_on(domain, lower, middle, upper) and is_bounded_below_on(domain, lower) and
    is_bounded_above_on(domain, upper)
    implies is_bounded_on(domain, middle)
}

/// The lower function in a pointwise between relation is bounded under endpoint bounds.
theorem bounded_on_lower_of_function_between_on[Domain, Value: PartialOrder](
    domain: Set[Domain], lower: Domain -> Value, middle: Domain -> Value, upper: Domain -> Value
) {
    function_between_on(domain, lower, middle, upper) and is_bounded_below_on(domain, lower) and
    is_bounded_above_on(domain, upper)
    implies is_bounded_on(domain, lower)
}

/// The upper function in a pointwise between relation is bounded under endpoint bounds.
theorem bounded_on_upper_of_function_between_on[Domain, Value: PartialOrder](
    domain: Set[Domain], lower: Domain -> Value, middle: Domain -> Value, upper: Domain -> Value
) {
    function_between_on(domain, lower, middle, upper) and is_bounded_below_on(domain, lower) and
    is_bounded_above_on(domain, upper)
    implies is_bounded_on(domain, upper)
}

/// All three functions in a pointwise between relation are bounded under endpoint bounds.
theorem bounded_on_triple_of_function_between_on[Domain, Value: PartialOrder](
    domain: Set[Domain], lower: Domain -> Value, middle: Domain -> Value, upper: Domain -> Value
) {
    function_between_on(domain, lower, middle, upper) and is_bounded_below_on(domain, lower) and
    is_bounded_above_on(domain, upper)
    implies is_bounded_on(domain, lower) and is_bounded_on(domain, middle) and is_bounded_on(domain, upper)
}

/// The middle of a strict pointwise between relation is bounded above when the upper function is bounded above.
theorem bounded_above_on_middle_of_function_strict_between_on[Domain, Value: PartialOrder](
    domain: Set[Domain], lower: Domain -> Value, middle: Domain -> Value, upper: Domain -> Value
) {
    function_strict_between_on(domain, lower, middle, upper) and is_bounded_above_on(domain, upper)
    implies is_bounded_above_on(domain, middle)
}

/// The middle of a strict pointwise between relation is bounded below when the lower function is bounded below.
theorem bounded_below_on_middle_of_function_strict_between_on[Domain, Value: PartialOrder](
    domain: Set[Domain], lower: Domain -> Value, middle: Domain -> Value, upper: Domain -> Value
) {
    function_strict_between_on(domain, lower, middle, upper) and is_bounded_below_on(domain, lower)
    implies is_bounded_below_on(domain, middle)
}

/// The lower function in a strict pointwise between relation is bounded above when the upper function is bounded above.
theorem bounded_above_on_lower_of_function_strict_between_on[Domain, Value: PartialOrder](
    domain: Set[Domain], lower: Domain -> Value, middle: Domain -> Value, upper: Domain -> Value
) {
    function_strict_between_on(domain, lower, middle, upper) and is_bounded_above_on(domain, upper)
    implies is_bounded_above_on(domain, lower)
}

/// The upper function in a strict pointwise between relation is bounded below when the lower function is bounded below.
theorem bounded_below_on_upper_of_function_strict_between_on[Domain, Value: PartialOrder](
    domain: Set[Domain], lower: Domain -> Value, middle: Domain -> Value, upper: Domain -> Value
) {
    function_strict_between_on(domain, lower, middle, upper) and is_bounded_below_on(domain, lower)
    implies is_bounded_below_on(domain, upper)
}

/// The middle of a strict pointwise between relation is bounded under endpoint bounds.
theorem bounded_on_middle_of_function_strict_between_on[Domain, Value: PartialOrder](
    domain: Set[Domain], lower: Domain -> Value, middle: Domain -> Value, upper: Domain -> Value
) {
    function_strict_between_on(domain, lower, middle, upper) and is_bounded_below_on(domain, lower) and
    is_bounded_above_on(domain, upper)
    implies is_bounded_on(domain, middle)
}

/// The lower function in a strict pointwise between relation is bounded under endpoint bounds.
theorem bounded_on_lower_of_function_strict_between_on[Domain, Value: PartialOrder](
    domain: Set[Domain], lower: Domain -> Value, middle: Domain -> Value, upper: Domain -> Value
) {
    function_strict_between_on(domain, lower, middle, upper) and is_bounded_below_on(domain, lower) and
    is_bounded_above_on(domain, upper)
    implies is_bounded_on(domain, lower)
}

/// The upper function in a strict pointwise between relation is bounded under endpoint bounds.
theorem bounded_on_upper_of_function_strict_between_on[Domain, Value: PartialOrder](
    domain: Set[Domain], lower: Domain -> Value, middle: Domain -> Value, upper: Domain -> Value
) {
    function_strict_between_on(domain, lower, middle, upper) and is_bounded_below_on(domain, lower) and
    is_bounded_above_on(domain, upper)
    implies is_bounded_on(domain, upper)
}

/// All three functions in a strict pointwise between relation are bounded under endpoint bounds.
theorem bounded_on_triple_of_function_strict_between_on[Domain, Value: PartialOrder](
    domain: Set[Domain], lower: Domain -> Value, middle: Domain -> Value, upper: Domain -> Value
) {
    function_strict_between_on(domain, lower, middle, upper) and is_bounded_below_on(domain, lower) and
    is_bounded_above_on(domain, upper)
    implies is_bounded_on(domain, lower) and is_bounded_on(domain, middle) and is_bounded_on(domain, upper)
}

/// An upper bound is preserved when the bounded function is replaced by an equal function on the domain.
theorem upper_bound_on_of_function_eq_on[Domain, Value: PartialOrder](
    domain: Set[Domain], original: Domain -> Value, replacement: Domain -> Value, bound: Value
) {
    function_eq_on(domain, original, replacement) and is_upper_bound_on(domain, original, bound)
    implies is_upper_bound_on(domain, replacement, bound)
}

/// A lower bound is preserved when the bounded function is replaced by an equal function on the domain.
theorem lower_bound_on_of_function_eq_on[Domain, Value: PartialOrder](
    domain: Set[Domain], original: Domain -> Value, replacement: Domain -> Value, bound: Value
) {
    function_eq_on(domain, original, replacement) and is_lower_bound_on(domain, original, bound)
    implies is_lower_bound_on(domain, replacement, bound)
}

/// An upper bound is reflected through equality on the domain.
theorem upper_bound_on_reflect_function_eq_on[Domain, Value: PartialOrder](
    domain: Set[Domain], original: Domain -> Value, replacement: Domain -> Value, bound: Value
) {
    function_eq_on(domain, original, replacement) and is_upper_bound_on(domain, replacement, bound)
    implies is_upper_bound_on(domain, original, bound)
}

/// A lower bound is reflected through equality on the domain.
theorem lower_bound_on_reflect_function_eq_on[Domain, Value: PartialOrder](
    domain: Set[Domain], original: Domain -> Value, replacement: Domain -> Value, bound: Value
) {
    function_eq_on(domain, original, replacement) and is_lower_bound_on(domain, replacement, bound)
    implies is_lower_bound_on(domain, original, bound)
}

/// Boundedness above is preserved when replacing a function by an equal function on the domain.
theorem bounded_above_on_of_function_eq_on[Domain, Value: PartialOrder](
    domain: Set[Domain], original: Domain -> Value, replacement: Domain -> Value
) {
    function_eq_on(domain, original, replacement) and is_bounded_above_on(domain, original)
    implies is_bounded_above_on(domain, replacement)
}

/// Boundedness below is preserved when replacing a function by an equal function on the domain.
theorem bounded_below_on_of_function_eq_on[Domain, Value: PartialOrder](
    domain: Set[Domain], original: Domain -> Value, replacement: Domain -> Value
) {
    function_eq_on(domain, original, replacement) and is_bounded_below_on(domain, original)
    implies is_bounded_below_on(domain, replacement)
}

/// Boundedness is preserved when replacing a function by an equal function on the domain.
theorem bounded_on_of_function_eq_on[Domain, Value: PartialOrder](
    domain: Set[Domain], original: Domain -> Value, replacement: Domain -> Value
) {
    function_eq_on(domain, original, replacement) and is_bounded_on(domain, original)
    implies is_bounded_on(domain, replacement)
}

/// Boundedness above is reflected through equality on the domain.
theorem bounded_above_on_reflect_function_eq_on[Domain, Value: PartialOrder](
    domain: Set[Domain], original: Domain -> Value, replacement: Domain -> Value
) {
    function_eq_on(domain, original, replacement) and is_bounded_above_on(domain, replacement)
    implies is_bounded_above_on(domain, original)
}

/// Boundedness below is reflected through equality on the domain.
theorem bounded_below_on_reflect_function_eq_on[Domain, Value: PartialOrder](
    domain: Set[Domain], original: Domain -> Value, replacement: Domain -> Value
) {
    function_eq_on(domain, original, replacement) and is_bounded_below_on(domain, replacement)
    implies is_bounded_below_on(domain, original)
}

/// Boundedness is reflected through equality on the domain.
theorem bounded_on_reflect_function_eq_on[Domain, Value: PartialOrder](
    domain: Set[Domain], original: Domain -> Value, replacement: Domain -> Value
) {
    function_eq_on(domain, original, replacement) and is_bounded_on(domain, replacement)
    implies is_bounded_on(domain, original)
}

/// A maximum point is preserved when replacing a function by an equal function on the domain.
theorem attains_maximum_at_of_function_eq_on[Domain, Value: PartialOrder](
    domain: Set[Domain], original: Domain -> Value, replacement: Domain -> Value, point: Domain
) {
    function_eq_on(domain, original, replacement) and attains_maximum_at(domain, original, point)
    implies attains_maximum_at(domain, replacement, point)
}

/// A minimum point is preserved when replacing a function by an equal function on the domain.
theorem attains_minimum_at_of_function_eq_on[Domain, Value: PartialOrder](
    domain: Set[Domain], original: Domain -> Value, replacement: Domain -> Value, point: Domain
) {
    function_eq_on(domain, original, replacement) and attains_minimum_at(domain, original, point)
    implies attains_minimum_at(domain, replacement, point)
}

/// A maximum point is reflected through equality on the domain.
theorem attains_maximum_at_reflect_function_eq_on[Domain, Value: PartialOrder](
    domain: Set[Domain], original: Domain -> Value, replacement: Domain -> Value, point: Domain
) {
    function_eq_on(domain, original, replacement) and attains_maximum_at(domain, replacement, point)
    implies attains_maximum_at(domain, original, point)
}

/// A minimum point is reflected through equality on the domain.
theorem attains_minimum_at_reflect_function_eq_on[Domain, Value: PartialOrder](
    domain: Set[Domain], original: Domain -> Value, replacement: Domain -> Value, point: Domain
) {
    function_eq_on(domain, original, replacement) and attains_minimum_at(domain, replacement, point)
    implies attains_minimum_at(domain, original, point)
}

/// Attainment of a maximum is preserved when replacing a function by an equal function on the domain.
theorem attains_maximum_on_of_function_eq_on[Domain, Value: PartialOrder](
    domain: Set[Domain], original: Domain -> Value, replacement: Domain -> Value
) {
    function_eq_on(domain, original, replacement) and attains_maximum_on(domain, original)
    implies attains_maximum_on(domain, replacement)
}

/// Attainment of a minimum is preserved when replacing a function by an equal function on the domain.
theorem attains_minimum_on_of_function_eq_on[Domain, Value: PartialOrder](
    domain: Set[Domain], original: Domain -> Value, replacement: Domain -> Value
) {
    function_eq_on(domain, original, replacement) and attains_minimum_on(domain, original)
    implies attains_minimum_on(domain, replacement)
}

/// Attainment of a maximum is reflected through equality on the domain.
theorem attains_maximum_on_reflect_function_eq_on[Domain, Value: PartialOrder](
    domain: Set[Domain], original: Domain -> Value, replacement: Domain -> Value
) {
    function_eq_on(domain, original, replacement) and attains_maximum_on(domain, replacement)
    implies attains_maximum_on(domain, original)
}

/// Attainment of a minimum is reflected through equality on the domain.
theorem attains_minimum_on_reflect_function_eq_on[Domain, Value: PartialOrder](
    domain: Set[Domain], original: Domain -> Value, replacement: Domain -> Value
) {
    function_eq_on(domain, original, replacement) and attains_minimum_on(domain, replacement)
    implies attains_minimum_on(domain, original)
}

// interval_set.ac
/// Intervals of a linear order as sets.

/// The closed interval `[lower, upper]` as a set.
define closed_interval_set[L: LinearOrder](lower: L, upper: L) -> Set[L] {
    Set[L].new(closed_interval(lower, upper))
}

/// The open interval `(lower, upper)` as a set.
define open_interval_set[L: LinearOrder](lower: L, upper: L) -> Set[L] {
    Set[L].new(open_interval(lower, upper))
}

/// The interval `(lower, upper]` as a set.
define left_open_interval_set[L: LinearOrder](lower: L, upper: L) -> Set[L] {
    Set[L].new(left_open_interval(lower, upper))
}

/// The interval `[lower, upper)` as a set.
define right_open_interval_set[L: LinearOrder](lower: L, upper: L) -> Set[L] {
    Set[L].new(right_open_interval(lower, upper))
}

/// Membership in a closed interval set is the closed interval predicate.
theorem closed_interval_set_contains_eq[L: LinearOrder](lower: L, upper: L, point: L) {
    closed_interval_set(lower, upper).contains(point) = closed_interval(lower, upper, point)
}

/// Membership in an open interval set is the open interval predicate.
theorem open_interval_set_contains_eq[L: LinearOrder](lower: L, upper: L, point: L) {
    open_interval_set(lower, upper).contains(point) = open_interval(lower, upper, point)
}

/// Membership in a left-open interval set is the left-open interval predicate.
theorem left_open_interval_set_contains_eq[L: LinearOrder](lower: L, upper: L, point: L) {
    left_open_interval_set(lower, upper).contains(point) = left_open_interval(lower, upper, point)
}

/// Membership in a right-open interval set is the right-open interval predicate.
theorem right_open_interval_set_contains_eq[L: LinearOrder](lower: L, upper: L, point: L) {
    right_open_interval_set(lower, upper).contains(point) = right_open_interval(lower, upper, point)
}

/// The lower endpoint belongs to a closed interval with ordered endpoints.
theorem closed_interval_set_contains_lower[L: LinearOrder](lower: L, upper: L) {
    lower <= upper implies closed_interval_set(lower, upper).contains(lower)
}

/// The upper endpoint belongs to a closed interval with ordered endpoints.
theorem closed_interval_set_contains_upper[L: LinearOrder](lower: L, upper: L) {
    lower <= upper implies closed_interval_set(lower, upper).contains(upper)
}

/// The lower endpoint inequality follows from membership in a closed interval set.
theorem closed_interval_set_lower_le[L: LinearOrder](lower: L, upper: L, point: L) {
    closed_interval_set(lower, upper).contains(point) implies lower <= point
}

/// The upper endpoint inequality follows from membership in a closed interval set.
theorem closed_interval_set_le_upper[L: LinearOrder](lower: L, upper: L, point: L) {
    closed_interval_set(lower, upper).contains(point) implies point <= upper
}

/// The right endpoint belongs to a left-open interval when the endpoints are strictly ordered.
theorem left_open_interval_set_contains_upper[L: LinearOrder](lower: L, upper: L) {
    lower < upper implies left_open_interval_set(lower, upper).contains(upper)
}

/// The left endpoint belongs to a right-open interval when the endpoints are strictly ordered.
theorem right_open_interval_set_contains_lower[L: LinearOrder](lower: L, upper: L) {
    lower < upper implies right_open_interval_set(lower, upper).contains(lower)
}

/// The lower strict endpoint inequality follows from membership in an open interval set.
theorem open_interval_set_lower_lt[L: LinearOrder](lower: L, upper: L, point: L) {
    open_interval_set(lower, upper).contains(point) implies lower < point
}

/// The upper strict endpoint inequality follows from membership in an open interval set.
theorem open_interval_set_lt_upper[L: LinearOrder](lower: L, upper: L, point: L) {
    open_interval_set(lower, upper).contains(point) implies point < upper
}

/// The lower strict endpoint inequality follows from membership in a left-open interval set.
theorem left_open_interval_set_lower_lt[L: LinearOrder](lower: L, upper: L, point: L) {
    left_open_interval_set(lower, upper).contains(point) implies lower < point
}

/// The upper endpoint inequality follows from membership in a left-open interval set.
theorem left_open_interval_set_le_upper[L: LinearOrder](lower: L, upper: L, point: L) {
    left_open_interval_set(lower, upper).contains(point) implies point <= upper
}

/// The lower endpoint inequality follows from membership in a right-open interval set.
theorem right_open_interval_set_lower_le[L: LinearOrder](lower: L, upper: L, point: L) {
    right_open_interval_set(lower, upper).contains(point) implies lower <= point
}

/// The upper strict endpoint inequality follows from membership in a right-open interval set.
theorem right_open_interval_set_lt_upper[L: LinearOrder](lower: L, upper: L, point: L) {
    right_open_interval_set(lower, upper).contains(point) implies point < upper
}

/// Every point of an open interval belongs to the corresponding closed interval.
theorem open_interval_set_subset_closed_interval_set[L: LinearOrder](lower: L, upper: L) {
    open_interval_set(lower, upper).subset(closed_interval_set(lower, upper))
}

/// Every point of a left-open interval belongs to the corresponding closed interval.
theorem left_open_interval_set_subset_closed_interval_set[L: LinearOrder](lower: L, upper: L) {
    left_open_interval_set(lower, upper).subset(closed_interval_set(lower, upper))
}

/// Every point of a right-open interval belongs to the corresponding closed interval.
theorem right_open_interval_set_subset_closed_interval_set[L: LinearOrder](lower: L, upper: L) {
    right_open_interval_set(lower, upper).subset(closed_interval_set(lower, upper))
}

/// Every point of an open interval belongs to the corresponding left-open interval.
theorem open_interval_set_subset_left_open_interval_set[L: LinearOrder](lower: L, upper: L) {
    open_interval_set(lower, upper).subset(left_open_interval_set(lower, upper))
}

/// Every point of an open interval belongs to the corresponding right-open interval.
theorem open_interval_set_subset_right_open_interval_set[L: LinearOrder](lower: L, upper: L) {
    open_interval_set(lower, upper).subset(right_open_interval_set(lower, upper))
}

/// A closed interval is contained in any closed interval with weaker endpoint bounds.
theorem closed_interval_set_subset_of_bounds[L: LinearOrder](
    outer_lower: L, lower: L, upper: L, outer_upper: L
) {
    outer_lower <= lower and upper <= outer_upper
    implies closed_interval_set(lower, upper).subset(closed_interval_set(outer_lower, outer_upper))
}

/// An open interval is contained in any open interval with weaker endpoint bounds.
theorem open_interval_set_subset_of_bounds[L: LinearOrder](
    outer_lower: L, lower: L, upper: L, outer_upper: L
) {
    outer_lower <= lower and upper <= outer_upper
    implies open_interval_set(lower, upper).subset(open_interval_set(outer_lower, outer_upper))
}

/// A left-open interval is contained in any left-open interval with weaker endpoint bounds.
theorem left_open_interval_set_subset_of_bounds[L: LinearOrder](
    outer_lower: L, lower: L, upper: L, outer_upper: L
) {
    outer_lower <= lower and upper <= outer_upper
    implies left_open_interval_set(lower, upper).subset(left_open_interval_set(outer_lower, outer_upper))
}

/// A right-open interval is contained in any right-open interval with weaker endpoint bounds.
theorem right_open_interval_set_subset_of_bounds[L: LinearOrder](
    outer_lower: L, lower: L, upper: L, outer_upper: L
) {
    outer_lower <= lower and upper <= outer_upper
    implies right_open_interval_set(lower, upper).subset(right_open_interval_set(outer_lower, outer_upper))
}

/// An open interval is contained in any closed interval with weaker endpoint bounds.
theorem open_interval_set_subset_closed_interval_set_of_bounds[L: LinearOrder](
    outer_lower: L, lower: L, upper: L, outer_upper: L
) {
    outer_lower <= lower and upper <= outer_upper
    implies open_interval_set(lower, upper).subset(closed_interval_set(outer_lower, outer_upper))
}

/// An open interval is contained in any left-open interval with weaker endpoint bounds.
theorem open_interval_set_subset_left_open_interval_set_of_bounds[L: LinearOrder](
    outer_lower: L, lower: L, upper: L, outer_upper: L
) {
    outer_lower <= lower and upper <= outer_upper
    implies open_interval_set(lower, upper).subset(left_open_interval_set(outer_lower, outer_upper))
}

/// An open interval is contained in any right-open interval with weaker endpoint bounds.
theorem open_interval_set_subset_right_open_interval_set_of_bounds[L: LinearOrder](
    outer_lower: L, lower: L, upper: L, outer_upper: L
) {
    outer_lower <= lower and upper <= outer_upper
    implies open_interval_set(lower, upper).subset(right_open_interval_set(outer_lower, outer_upper))
}

/// A left-open interval is contained in any closed interval with weaker endpoint bounds.
theorem left_open_interval_set_subset_closed_interval_set_of_bounds[L: LinearOrder](
    outer_lower: L, lower: L, upper: L, outer_upper: L
) {
    outer_lower <= lower and upper <= outer_upper
    implies left_open_interval_set(lower, upper).subset(closed_interval_set(outer_lower, outer_upper))
}

/// A right-open interval is contained in any closed interval with weaker endpoint bounds.
theorem right_open_interval_set_subset_closed_interval_set_of_bounds[L: LinearOrder](
    outer_lower: L, lower: L, upper: L, outer_upper: L
) {
    outer_lower <= lower and upper <= outer_upper
    implies right_open_interval_set(lower, upper).subset(closed_interval_set(outer_lower, outer_upper))
}

/// A left-open interval is contained in a right-open interval with a strictly weaker upper bound.
theorem left_open_interval_set_subset_right_open_interval_set_of_upper_strict_bound[L: LinearOrder](
    outer_lower: L, lower: L, upper: L, outer_upper: L
) {
    outer_lower <= lower and upper < outer_upper
    implies left_open_interval_set(lower, upper).subset(right_open_interval_set(outer_lower, outer_upper))
}

/// A right-open interval is contained in a left-open interval with a strictly weaker lower bound.
theorem right_open_interval_set_subset_left_open_interval_set_of_lower_strict_bound[L: LinearOrder](
    outer_lower: L, lower: L, upper: L, outer_upper: L
) {
    outer_lower < lower and upper <= outer_upper
    implies right_open_interval_set(lower, upper).subset(left_open_interval_set(outer_lower, outer_upper))
}

/// A closed interval is contained in an open interval whose bounds are strictly weaker.
theorem closed_interval_set_subset_open_interval_set_of_strict_bounds[L: LinearOrder](
    outer_lower: L, lower: L, upper: L, outer_upper: L
) {
    outer_lower < lower and upper < outer_upper
    implies closed_interval_set(lower, upper).subset(open_interval_set(outer_lower, outer_upper))
}

/// A closed interval is contained in a left-open interval with a strictly weaker lower bound.
theorem closed_interval_set_subset_left_open_interval_set_of_lower_strict_bound[L: LinearOrder](
    outer_lower: L, lower: L, upper: L, outer_upper: L
) {
    outer_lower < lower and upper <= outer_upper
    implies closed_interval_set(lower, upper).subset(left_open_interval_set(outer_lower, outer_upper))
}

/// A closed interval is contained in a right-open interval with a strictly weaker upper bound.
theorem closed_interval_set_subset_right_open_interval_set_of_upper_strict_bound[L: LinearOrder](
    outer_lower: L, lower: L, upper: L, outer_upper: L
) {
    outer_lower <= lower and upper < outer_upper
    implies closed_interval_set(lower, upper).subset(right_open_interval_set(outer_lower, outer_upper))
}

/// A left-open interval is contained in an open interval with a strictly weaker upper bound.
theorem left_open_interval_set_subset_open_interval_set_of_upper_strict_bound[L: LinearOrder](
    outer_lower: L, lower: L, upper: L, outer_upper: L
) {
    outer_lower <= lower and upper < outer_upper
    implies left_open_interval_set(lower, upper).subset(open_interval_set(outer_lower, outer_upper))
}

/// A right-open interval is contained in an open interval with a strictly weaker lower bound.
theorem right_open_interval_set_subset_open_interval_set_of_lower_strict_bound[L: LinearOrder](
    outer_lower: L, lower: L, upper: L, outer_upper: L
) {
    outer_lower < lower and upper <= outer_upper
    implies right_open_interval_set(lower, upper).subset(open_interval_set(outer_lower, outer_upper))
}

/// The intersection of two closed interval sets is the closed interval set with joined lower endpoints and met upper endpoints.
theorem closed_interval_set_intersection_eq[L: LinearOrder](a1: L, b1: L, a2: L, b2: L) {
    closed_interval_set(a1, b1).intersection(closed_interval_set(a2, b2)) =
    closed_interval_set(a1.max(a2), b1.min(b2))
}

/// The intersection of two open interval sets is the open interval set with joined lower endpoints and met upper endpoints.
theorem open_interval_set_intersection_eq[L: LinearOrder](a1: L, b1: L, a2: L, b2: L) {
    open_interval_set(a1, b1).intersection(open_interval_set(a2, b2)) =
    open_interval_set(a1.max(a2), b1.min(b2))
}

/// The intersection of two left-open interval sets is the left-open interval set with joined lower endpoints and met upper endpoints.
theorem left_open_interval_set_intersection_eq[L: LinearOrder](a1: L, b1: L, a2: L, b2: L) {
    left_open_interval_set(a1, b1).intersection(left_open_interval_set(a2, b2)) =
    left_open_interval_set(a1.max(a2), b1.min(b2))
}

/// The intersection of two right-open interval sets is the right-open interval set with joined lower endpoints and met upper endpoints.
theorem right_open_interval_set_intersection_eq[L: LinearOrder](a1: L, b1: L, a2: L, b2: L) {
    right_open_interval_set(a1, b1).intersection(right_open_interval_set(a2, b2)) =
    right_open_interval_set(a1.max(a2), b1.min(b2))
}

// maps_on.ac
/// Monotone and antitone maps restricted to a subset of their domain.

/// True if `f` preserves the non-strict order on `domain`.
define is_monotone_on[Domain: PartialOrder, Value: PartialOrder](
    domain: Set[Domain], f: Domain -> Value
) -> Bool {
    forall(left: Domain, right: Domain) {
        domain.contains(left) and domain.contains(right) and left <= right implies f(left) <= f(right)
    }
}

/// True if `f` reverses the non-strict order on `domain`.
define is_antitone_on[Domain: PartialOrder, Value: PartialOrder](
    domain: Set[Domain], f: Domain -> Value
) -> Bool {
    forall(left: Domain, right: Domain) {
        domain.contains(left) and domain.contains(right) and left <= right implies f(right) <= f(left)
    }
}

/// True if `f` preserves strict order on `domain`.
define is_strict_monotone_on[Domain: PartialOrder, Value: PartialOrder](
    domain: Set[Domain], f: Domain -> Value
) -> Bool {
    forall(left: Domain, right: Domain) {
        domain.contains(left) and domain.contains(right) and left < right implies f(left) < f(right)
    }
}

/// True if `f` reverses strict order on `domain`.
define is_strict_antitone_on[Domain: PartialOrder, Value: PartialOrder](
    domain: Set[Domain], f: Domain -> Value
) -> Bool {
    forall(left: Domain, right: Domain) {
        domain.contains(left) and domain.contains(right) and left < right implies f(right) < f(left)
    }
}

/// A monotone map on a set carries an ordered pair of points to an ordered pair of values.
theorem monotone_on_apply[Domain: PartialOrder, Value: PartialOrder](
    domain: Set[Domain], f: Domain -> Value, left: Domain, right: Domain
) {
    is_monotone_on(domain, f) and domain.contains(left) and domain.contains(right) and left <= right
    implies f(left) <= f(right)
}

/// An antitone map on a set reverses an ordered pair of points.
theorem antitone_on_apply[Domain: PartialOrder, Value: PartialOrder](
    domain: Set[Domain], f: Domain -> Value, left: Domain, right: Domain
) {
    is_antitone_on(domain, f) and domain.contains(left) and domain.contains(right) and left <= right
    implies f(right) <= f(left)
}

/// A strictly monotone map on a set carries a strictly ordered pair to a strictly ordered pair.
theorem strict_monotone_on_apply[Domain: PartialOrder, Value: PartialOrder](
    domain: Set[Domain], f: Domain -> Value, left: Domain, right: Domain
) {
    is_strict_monotone_on(domain, f) and domain.contains(left) and domain.contains(right) and left < right
    implies f(left) < f(right)
}

/// A strictly antitone map on a set reverses a strictly ordered pair.
theorem strict_antitone_on_apply[Domain: PartialOrder, Value: PartialOrder](
    domain: Set[Domain], f: Domain -> Value, left: Domain, right: Domain
) {
    is_strict_antitone_on(domain, f) and domain.contains(left) and domain.contains(right) and left < right
    implies f(right) < f(left)
}

/// Monotonicity on a set is preserved by pointwise equality on that set.
theorem monotone_on_of_function_eq_on[Domain: PartialOrder, Value: PartialOrder](
    domain: Set[Domain], original: Domain -> Value, replacement: Domain -> Value
) {
    function_eq_on(domain, original, replacement) and is_monotone_on(domain, original)
    implies is_monotone_on(domain, replacement)
}

/// Antitonicity on a set is preserved by pointwise equality on that set.
theorem antitone_on_of_function_eq_on[Domain: PartialOrder, Value: PartialOrder](
    domain: Set[Domain], original: Domain -> Value, replacement: Domain -> Value
) {
    function_eq_on(domain, original, replacement) and is_antitone_on(domain, original)
    implies is_antitone_on(domain, replacement)
}

/// Strict monotonicity on a set is preserved by pointwise equality on that set.
theorem strict_monotone_on_of_function_eq_on[Domain: PartialOrder, Value: PartialOrder](
    domain: Set[Domain], original: Domain -> Value, replacement: Domain -> Value
) {
    function_eq_on(domain, original, replacement) and is_strict_monotone_on(domain, original)
    implies is_strict_monotone_on(domain, replacement)
}

/// Strict antitonicity on a set is preserved by pointwise equality on that set.
theorem strict_antitone_on_of_function_eq_on[Domain: PartialOrder, Value: PartialOrder](
    domain: Set[Domain], original: Domain -> Value, replacement: Domain -> Value
) {
    function_eq_on(domain, original, replacement) and is_strict_antitone_on(domain, original)
    implies is_strict_antitone_on(domain, replacement)
}

/// Monotonicity on a set is reflected by pointwise equality on that set.
theorem monotone_on_reflect_function_eq_on[Domain: PartialOrder, Value: PartialOrder](
    domain: Set[Domain], original: Domain -> Value, replacement: Domain -> Value
) {
    function_eq_on(domain, original, replacement) and is_monotone_on(domain, replacement)
    implies is_monotone_on(domain, original)
}

/// Antitonicity on a set is reflected by pointwise equality on that set.
theorem antitone_on_reflect_function_eq_on[Domain: PartialOrder, Value: PartialOrder](
    domain: Set[Domain], original: Domain -> Value, replacement: Domain -> Value
) {
    function_eq_on(domain, original, replacement) and is_antitone_on(domain, replacement)
    implies is_antitone_on(domain, original)
}

/// Strict monotonicity on a set is reflected by pointwise equality on that set.
theorem strict_monotone_on_reflect_function_eq_on[Domain: PartialOrder, Value: PartialOrder](
    domain: Set[Domain], original: Domain -> Value, replacement: Domain -> Value
) {
    function_eq_on(domain, original, replacement) and is_strict_monotone_on(domain, replacement)
    implies is_strict_monotone_on(domain, original)
}

/// Strict antitonicity on a set is reflected by pointwise equality on that set.
theorem strict_antitone_on_reflect_function_eq_on[Domain: PartialOrder, Value: PartialOrder](
    domain: Set[Domain], original: Domain -> Value, replacement: Domain -> Value
) {
    function_eq_on(domain, original, replacement) and is_strict_antitone_on(domain, replacement)
    implies is_strict_antitone_on(domain, original)
}

/// A globally monotone map is monotone on every subset.
theorem monotone_on_of_monotone[Domain: PartialOrder, Value: PartialOrder](
    domain: Set[Domain], f: Domain -> Value
) {
    is_monotone(f) implies is_monotone_on(domain, f)
}

/// A globally antitone map is antitone on every subset.
theorem antitone_on_of_antitone[Domain: PartialOrder, Value: PartialOrder](
    domain: Set[Domain], f: Domain -> Value
) {
    is_antitone(f) implies is_antitone_on(domain, f)
}

/// A globally strictly monotone map is strictly monotone on every subset.
theorem strict_monotone_on_of_strict_monotone[Domain: PartialOrder, Value: PartialOrder](
    domain: Set[Domain], f: Domain -> Value
) {
    is_strict_monotone(f) implies is_strict_monotone_on(domain, f)
}

/// A globally strictly antitone map is strictly antitone on every subset.
theorem strict_antitone_on_of_strict_antitone[Domain: PartialOrder, Value: PartialOrder](
    domain: Set[Domain], f: Domain -> Value
) {
    is_strict_antitone(f) implies is_strict_antitone_on(domain, f)
}

/// A globally monotone map remains monotone on a set after replacement by an equal function there.
theorem monotone_on_of_monotone_and_function_eq_on[Domain: PartialOrder, Value: PartialOrder](
    domain: Set[Domain], original: Domain -> Value, replacement: Domain -> Value
) {
    is_monotone(original) and function_eq_on(domain, original, replacement)
    implies is_monotone_on(domain, replacement)
}

/// A globally antitone map remains antitone on a set after replacement by an equal function there.
theorem antitone_on_of_antitone_and_function_eq_on[Domain: PartialOrder, Value: PartialOrder](
    domain: Set[Domain], original: Domain -> Value, replacement: Domain -> Value
) {
    is_antitone(original) and function_eq_on(domain, original, replacement)
    implies is_antitone_on(domain, replacement)
}

/// A globally strictly monotone map remains strictly monotone on a set after replacement by an equal function there.
theorem strict_monotone_on_of_strict_monotone_and_function_eq_on[Domain: PartialOrder, Value: PartialOrder](
    domain: Set[Domain], original: Domain -> Value, replacement: Domain -> Value
) {
    is_strict_monotone(original) and function_eq_on(domain, original, replacement)
    implies is_strict_monotone_on(domain, replacement)
}

/// A globally strictly antitone map remains strictly antitone on a set after replacement by an equal function there.
theorem strict_antitone_on_of_strict_antitone_and_function_eq_on[Domain: PartialOrder, Value: PartialOrder](
    domain: Set[Domain], original: Domain -> Value, replacement: Domain -> Value
) {
    is_strict_antitone(original) and function_eq_on(domain, original, replacement)
    implies is_strict_antitone_on(domain, replacement)
}

/// Both pointwise-equal functions are monotone when one is monotone on the set.
theorem monotone_on_pair_of_function_eq_on[Domain: PartialOrder, Value: PartialOrder](
    domain: Set[Domain], left: Domain -> Value, right: Domain -> Value
) {
    function_eq_on(domain, left, right) and is_monotone_on(domain, left)
    implies is_monotone_on(domain, left) and is_monotone_on(domain, right)
}

/// Both pointwise-equal functions are antitone when one is antitone on the set.
theorem antitone_on_pair_of_function_eq_on[Domain: PartialOrder, Value: PartialOrder](
    domain: Set[Domain], left: Domain -> Value, right: Domain -> Value
) {
    function_eq_on(domain, left, right) and is_antitone_on(domain, left)
    implies is_antitone_on(domain, left) and is_antitone_on(domain, right)
}

/// Both pointwise-equal functions are strictly monotone when one is strictly monotone on the set.
theorem strict_monotone_on_pair_of_function_eq_on[Domain: PartialOrder, Value: PartialOrder](
    domain: Set[Domain], left: Domain -> Value, right: Domain -> Value
) {
    function_eq_on(domain, left, right) and is_strict_monotone_on(domain, left)
    implies is_strict_monotone_on(domain, left) and is_strict_monotone_on(domain, right)
}

/// Both pointwise-equal functions are strictly antitone when one is strictly antitone on the set.
theorem strict_antitone_on_pair_of_function_eq_on[Domain: PartialOrder, Value: PartialOrder](
    domain: Set[Domain], left: Domain -> Value, right: Domain -> Value
) {
    function_eq_on(domain, left, right) and is_strict_antitone_on(domain, left)
    implies is_strict_antitone_on(domain, left) and is_strict_antitone_on(domain, right)
}

/// The identity map is monotone on every subset of a partial order.
theorem identity_monotone_on[Domain: PartialOrder](domain: Set[Domain]) {
    is_monotone_on(domain, identity_fn[Domain])
}

/// The identity map is strictly monotone on every subset of a partial order.
theorem identity_strict_monotone_on[Domain: PartialOrder](domain: Set[Domain]) {
    is_strict_monotone_on(domain, identity_fn[Domain])
}

/// A constant-valued map is monotone on every subset.
theorem constant_monotone_on[Domain: PartialOrder, Value: PartialOrder](
    domain: Set[Domain], value: Value
) {
    is_monotone_on(domain, constant[Domain, Value](value))
}

/// A constant-valued map is antitone on every subset.
theorem constant_antitone_on[Domain: PartialOrder, Value: PartialOrder](
    domain: Set[Domain], value: Value
) {
    is_antitone_on(domain, constant[Domain, Value](value))
}

/// Composing a monotone map after a monotone map on a set gives a monotone map on that set.
theorem monotone_compose_monotone_on[Domain: PartialOrder, Middle: PartialOrder, Value: PartialOrder](
    domain: Set[Domain], outer: Middle -> Value, inner: Domain -> Middle
) {
    is_monotone(outer) and is_monotone_on(domain, inner) implies is_monotone_on(domain, compose(outer, inner))
}

/// Composing an antitone map after a monotone map on a set gives an antitone map on that set.
theorem antitone_compose_monotone_on[Domain: PartialOrder, Middle: PartialOrder, Value: PartialOrder](
    domain: Set[Domain], outer: Middle -> Value, inner: Domain -> Middle
) {
    is_antitone(outer) and is_monotone_on(domain, inner) implies is_antitone_on(domain, compose(outer, inner))
}

/// Composing a monotone map after an antitone map on a set gives an antitone map on that set.
theorem monotone_compose_antitone_on[Domain: PartialOrder, Middle: PartialOrder, Value: PartialOrder](
    domain: Set[Domain], outer: Middle -> Value, inner: Domain -> Middle
) {
    is_monotone(outer) and is_antitone_on(domain, inner) implies is_antitone_on(domain, compose(outer, inner))
}

/// Composing an antitone map after an antitone map on a set gives a monotone map on that set.
theorem antitone_compose_antitone_on[Domain: PartialOrder, Middle: PartialOrder, Value: PartialOrder](
    domain: Set[Domain], outer: Middle -> Value, inner: Domain -> Middle
) {
    is_antitone(outer) and is_antitone_on(domain, inner) implies is_monotone_on(domain, compose(outer, inner))
}

/// Composing a strictly monotone map after a strictly monotone map on a set gives a strictly monotone map.
theorem strict_monotone_compose_strict_monotone_on[Domain: PartialOrder, Middle: PartialOrder, Value: PartialOrder](
    domain: Set[Domain], outer: Middle -> Value, inner: Domain -> Middle
) {
    is_strict_monotone(outer) and is_strict_monotone_on(domain, inner)
    implies is_strict_monotone_on(domain, compose(outer, inner))
}

/// Composing a strictly antitone map after a strictly monotone map on a set gives a strictly antitone map.
theorem strict_antitone_compose_strict_monotone_on[Domain: PartialOrder, Middle: PartialOrder, Value: PartialOrder](
    domain: Set[Domain], outer: Middle -> Value, inner: Domain -> Middle
) {
    is_strict_antitone(outer) and is_strict_monotone_on(domain, inner)
    implies is_strict_antitone_on(domain, compose(outer, inner))
}

/// Composing a strictly monotone map after a strictly antitone map on a set gives a strictly antitone map.
theorem strict_monotone_compose_strict_antitone_on[Domain: PartialOrder, Middle: PartialOrder, Value: PartialOrder](
    domain: Set[Domain], outer: Middle -> Value, inner: Domain -> Middle
) {
    is_strict_monotone(outer) and is_strict_antitone_on(domain, inner)
    implies is_strict_antitone_on(domain, compose(outer, inner))
}

/// Composing a strictly antitone map after a strictly antitone map on a set gives a strictly monotone map.
theorem strict_antitone_compose_strict_antitone_on[Domain: PartialOrder, Middle: PartialOrder, Value: PartialOrder](
    domain: Set[Domain], outer: Middle -> Value, inner: Domain -> Middle
) {
    is_strict_antitone(outer) and is_strict_antitone_on(domain, inner)
    implies is_strict_monotone_on(domain, compose(outer, inner))
}

/// Composing a monotone map on the image after a monotone map on a set gives a monotone map on that set.
theorem monotone_on_compose_monotone_on_image[Domain: PartialOrder, Middle: PartialOrder, Value: PartialOrder](
    domain: Set[Domain], outer: Middle -> Value, inner: Domain -> Middle
) {
    is_monotone_on(set_image(domain, inner), outer) and is_monotone_on(domain, inner)
    implies is_monotone_on(domain, compose(outer, inner))
}

/// Composing an antitone map on the image after a monotone map on a set gives an antitone map on that set.
theorem antitone_on_compose_monotone_on_image[Domain: PartialOrder, Middle: PartialOrder, Value: PartialOrder](
    domain: Set[Domain], outer: Middle -> Value, inner: Domain -> Middle
) {
    is_antitone_on(set_image(domain, inner), outer) and is_monotone_on(domain, inner)
    implies is_antitone_on(domain, compose(outer, inner))
}

/// Composing a monotone map on the image after an antitone map on a set gives an antitone map on that set.
theorem monotone_on_compose_antitone_on_image[Domain: PartialOrder, Middle: PartialOrder, Value: PartialOrder](
    domain: Set[Domain], outer: Middle -> Value, inner: Domain -> Middle
) {
    is_monotone_on(set_image(domain, inner), outer) and is_antitone_on(domain, inner)
    implies is_antitone_on(domain, compose(outer, inner))
}

/// Composing an antitone map on the image after an antitone map on a set gives a monotone map on that set.
theorem antitone_on_compose_antitone_on_image[Domain: PartialOrder, Middle: PartialOrder, Value: PartialOrder](
    domain: Set[Domain], outer: Middle -> Value, inner: Domain -> Middle
) {
    is_antitone_on(set_image(domain, inner), outer) and is_antitone_on(domain, inner)
    implies is_monotone_on(domain, compose(outer, inner))
}

/// Composing a strictly monotone map on the image after a strictly monotone map gives a strictly monotone map.
theorem strict_monotone_on_compose_strict_monotone_on_image[Domain: PartialOrder, Middle: PartialOrder, Value: PartialOrder](
    domain: Set[Domain], outer: Middle -> Value, inner: Domain -> Middle
) {
    is_strict_monotone_on(set_image(domain, inner), outer) and is_strict_monotone_on(domain, inner)
    implies is_strict_monotone_on(domain, compose(outer, inner))
}

/// Composing a strictly antitone map on the image after a strictly monotone map gives a strictly antitone map.
theorem strict_antitone_on_compose_strict_monotone_on_image[Domain: PartialOrder, Middle: PartialOrder, Value: PartialOrder](
    domain: Set[Domain], outer: Middle -> Value, inner: Domain -> Middle
) {
    is_strict_antitone_on(set_image(domain, inner), outer) and is_strict_monotone_on(domain, inner)
    implies is_strict_antitone_on(domain, compose(outer, inner))
}

/// Composing a strictly monotone map on the image after a strictly antitone map gives a strictly antitone map.
theorem strict_monotone_on_compose_strict_antitone_on_image[Domain: PartialOrder, Middle: PartialOrder, Value: PartialOrder](
    domain: Set[Domain], outer: Middle -> Value, inner: Domain -> Middle
) {
    is_strict_monotone_on(set_image(domain, inner), outer) and is_strict_antitone_on(domain, inner)
    implies is_strict_antitone_on(domain, compose(outer, inner))
}

/// Composing a strictly antitone map on the image after a strictly antitone map gives a strictly monotone map.
theorem strict_antitone_on_compose_strict_antitone_on_image[Domain: PartialOrder, Middle: PartialOrder, Value: PartialOrder](
    domain: Set[Domain], outer: Middle -> Value, inner: Domain -> Middle
) {
    is_strict_antitone_on(set_image(domain, inner), outer) and is_strict_antitone_on(domain, inner)
    implies is_strict_monotone_on(domain, compose(outer, inner))
}

/// Strict monotonicity on a set implies non-strict monotonicity on that set.
theorem strict_monotone_on_imp_monotone_on[Domain: PartialOrder, Value: PartialOrder](
    domain: Set[Domain], f: Domain -> Value
) {
    is_strict_monotone_on(domain, f) implies is_monotone_on(domain, f)
}

/// Strict antitonicity on a set implies non-strict antitonicity on that set.
theorem strict_antitone_on_imp_antitone_on[Domain: PartialOrder, Value: PartialOrder](
    domain: Set[Domain], f: Domain -> Value
) {
    is_strict_antitone_on(domain, f) implies is_antitone_on(domain, f)
}

/// A function equal on the set to a strictly monotone function is monotone there.
theorem monotone_on_of_strict_monotone_on_and_function_eq_on[Domain: PartialOrder, Value: PartialOrder](
    domain: Set[Domain], original: Domain -> Value, replacement: Domain -> Value
) {
    is_strict_monotone_on(domain, original) and function_eq_on(domain, original, replacement)
    implies is_monotone_on(domain, replacement)
}

/// A function equal on the set to a strictly antitone function is antitone there.
theorem antitone_on_of_strict_antitone_on_and_function_eq_on[Domain: PartialOrder, Value: PartialOrder](
    domain: Set[Domain], original: Domain -> Value, replacement: Domain -> Value
) {
    is_strict_antitone_on(domain, original) and function_eq_on(domain, original, replacement)
    implies is_antitone_on(domain, replacement)
}

/// A function equal on the set to a globally strictly monotone function is monotone there.
theorem monotone_on_of_strict_monotone_and_function_eq_on[Domain: PartialOrder, Value: PartialOrder](
    domain: Set[Domain], original: Domain -> Value, replacement: Domain -> Value
) {
    is_strict_monotone(original) and function_eq_on(domain, original, replacement)
    implies is_monotone_on(domain, replacement)
}

/// A function equal on the set to a globally strictly antitone function is antitone there.
theorem antitone_on_of_strict_antitone_and_function_eq_on[Domain: PartialOrder, Value: PartialOrder](
    domain: Set[Domain], original: Domain -> Value, replacement: Domain -> Value
) {
    is_strict_antitone(original) and function_eq_on(domain, original, replacement)
    implies is_antitone_on(domain, replacement)
}

/// Monotonicity on a set is preserved by restricting the domain.
theorem monotone_on_subset[Domain: PartialOrder, Value: PartialOrder](
    subset: Set[Domain], superset: Set[Domain], f: Domain -> Value
) {
    subset.subset(superset) and is_monotone_on(superset, f) implies is_monotone_on(subset, f)
}

/// Antitonicity on a set is preserved by restricting the domain.
theorem antitone_on_subset[Domain: PartialOrder, Value: PartialOrder](
    subset: Set[Domain], superset: Set[Domain], f: Domain -> Value
) {
    subset.subset(superset) and is_antitone_on(superset, f) implies is_antitone_on(subset, f)
}

/// Strict monotonicity on a set is preserved by restricting the domain.
theorem strict_monotone_on_subset[Domain: PartialOrder, Value: PartialOrder](
    subset: Set[Domain], superset: Set[Domain], f: Domain -> Value
) {
    subset.subset(superset) and is_strict_monotone_on(superset, f) implies is_strict_monotone_on(subset, f)
}

/// Strict antitonicity on a set is preserved by restricting the domain.
theorem strict_antitone_on_subset[Domain: PartialOrder, Value: PartialOrder](
    subset: Set[Domain], superset: Set[Domain], f: Domain -> Value
) {
    subset.subset(superset) and is_strict_antitone_on(superset, f) implies is_strict_antitone_on(subset, f)
}

// monotone_interval_bounds.ac
/// Bounds for monotone maps on closed intervals.

/// A monotone map on `[lower, upper]` is bounded above by its value at `upper`.
theorem monotone_on_closed_interval_upper_bound[Domain: LinearOrder, Value: PartialOrder](
    lower: Domain, upper: Domain, f: Domain -> Value
) {
    lower <= upper and is_monotone_on(closed_interval_set(lower, upper), f)
    implies is_upper_bound_on(closed_interval_set(lower, upper), f, f(upper))
}

/// A monotone map on `[lower, upper]` is bounded below by its value at `lower`.
theorem monotone_on_closed_interval_lower_bound[Domain: LinearOrder, Value: PartialOrder](
    lower: Domain, upper: Domain, f: Domain -> Value
) {
    lower <= upper and is_monotone_on(closed_interval_set(lower, upper), f)
    implies is_lower_bound_on(closed_interval_set(lower, upper), f, f(lower))
}

/// A monotone map on `[lower, upper]` is bounded above.
theorem monotone_on_closed_interval_bounded_above[Domain: LinearOrder, Value: PartialOrder](
    lower: Domain, upper: Domain, f: Domain -> Value
) {
    lower <= upper and is_monotone_on(closed_interval_set(lower, upper), f)
    implies is_bounded_above_on(closed_interval_set(lower, upper), f)
}

/// A monotone map on `[lower, upper]` is bounded below.
theorem monotone_on_closed_interval_bounded_below[Domain: LinearOrder, Value: PartialOrder](
    lower: Domain, upper: Domain, f: Domain -> Value
) {
    lower <= upper and is_monotone_on(closed_interval_set(lower, upper), f)
    implies is_bounded_below_on(closed_interval_set(lower, upper), f)
}

/// A monotone map on `[lower, upper]` is bounded.
theorem monotone_on_closed_interval_bounded[Domain: LinearOrder, Value: PartialOrder](
    lower: Domain, upper: Domain, f: Domain -> Value
) {
    lower <= upper and is_monotone_on(closed_interval_set(lower, upper), f)
    implies is_bounded_on(closed_interval_set(lower, upper), f)
}

/// A monotone map on `[lower, upper]` is bounded above on `(lower, upper)` by its value at `upper`.
theorem monotone_on_open_interval_upper_bound_from_closed_interval[Domain: LinearOrder, Value: PartialOrder](
    lower: Domain, upper: Domain, f: Domain -> Value
) {
    lower <= upper and is_monotone_on(closed_interval_set(lower, upper), f)
    implies is_upper_bound_on(open_interval_set(lower, upper), f, f(upper))
}

/// A monotone map on `[lower, upper]` is bounded below on `(lower, upper)` by its value at `lower`.
theorem monotone_on_open_interval_lower_bound_from_closed_interval[Domain: LinearOrder, Value: PartialOrder](
    lower: Domain, upper: Domain, f: Domain -> Value
) {
    lower <= upper and is_monotone_on(closed_interval_set(lower, upper), f)
    implies is_lower_bound_on(open_interval_set(lower, upper), f, f(lower))
}

/// A monotone map on `[lower, upper]` is bounded above on `(lower, upper)`.
theorem monotone_on_open_interval_bounded_above_from_closed_interval[Domain: LinearOrder, Value: PartialOrder](
    lower: Domain, upper: Domain, f: Domain -> Value
) {
    lower <= upper and is_monotone_on(closed_interval_set(lower, upper), f)
    implies is_bounded_above_on(open_interval_set(lower, upper), f)
}

/// A monotone map on `[lower, upper]` is bounded below on `(lower, upper)`.
theorem monotone_on_open_interval_bounded_below_from_closed_interval[Domain: LinearOrder, Value: PartialOrder](
    lower: Domain, upper: Domain, f: Domain -> Value
) {
    lower <= upper and is_monotone_on(closed_interval_set(lower, upper), f)
    implies is_bounded_below_on(open_interval_set(lower, upper), f)
}

/// A monotone map on `[lower, upper]` is bounded on `(lower, upper)`.
theorem monotone_on_open_interval_bounded_from_closed_interval[Domain: LinearOrder, Value: PartialOrder](
    lower: Domain, upper: Domain, f: Domain -> Value
) {
    lower <= upper and is_monotone_on(closed_interval_set(lower, upper), f)
    implies is_bounded_on(open_interval_set(lower, upper), f)
}

/// A monotone map on `[lower, upper]` is bounded above on `(lower, upper]` by its value at `upper`.
theorem monotone_on_left_open_interval_upper_bound_from_closed_interval[Domain: LinearOrder, Value: PartialOrder](
    lower: Domain, upper: Domain, f: Domain -> Value
) {
    lower <= upper and is_monotone_on(closed_interval_set(lower, upper), f)
    implies is_upper_bound_on(left_open_interval_set(lower, upper), f, f(upper))
}

/// A monotone map on `[lower, upper]` is bounded below on `(lower, upper]` by its value at `lower`.
theorem monotone_on_left_open_interval_lower_bound_from_closed_interval[Domain: LinearOrder, Value: PartialOrder](
    lower: Domain, upper: Domain, f: Domain -> Value
) {
    lower <= upper and is_monotone_on(closed_interval_set(lower, upper), f)
    implies is_lower_bound_on(left_open_interval_set(lower, upper), f, f(lower))
}

/// A monotone map on `[lower, upper]` is bounded above on `(lower, upper]`.
theorem monotone_on_left_open_interval_bounded_above_from_closed_interval[Domain: LinearOrder, Value: PartialOrder](
    lower: Domain, upper: Domain, f: Domain -> Value
) {
    lower <= upper and is_monotone_on(closed_interval_set(lower, upper), f)
    implies is_bounded_above_on(left_open_interval_set(lower, upper), f)
}

/// A monotone map on `[lower, upper]` is bounded below on `(lower, upper]`.
theorem monotone_on_left_open_interval_bounded_below_from_closed_interval[Domain: LinearOrder, Value: PartialOrder](
    lower: Domain, upper: Domain, f: Domain -> Value
) {
    lower <= upper and is_monotone_on(closed_interval_set(lower, upper), f)
    implies is_bounded_below_on(left_open_interval_set(lower, upper), f)
}

/// A monotone map on `[lower, upper]` is bounded on `(lower, upper]`.
theorem monotone_on_left_open_interval_bounded_from_closed_interval[Domain: LinearOrder, Value: PartialOrder](
    lower: Domain, upper: Domain, f: Domain -> Value
) {
    lower <= upper and is_monotone_on(closed_interval_set(lower, upper), f)
    implies is_bounded_on(left_open_interval_set(lower, upper), f)
}

/// A monotone map on `[lower, upper]` is bounded above on `[lower, upper)` by its value at `upper`.
theorem monotone_on_right_open_interval_upper_bound_from_closed_interval[Domain: LinearOrder, Value: PartialOrder](
    lower: Domain, upper: Domain, f: Domain -> Value
) {
    lower <= upper and is_monotone_on(closed_interval_set(lower, upper), f)
    implies is_upper_bound_on(right_open_interval_set(lower, upper), f, f(upper))
}

/// A monotone map on `[lower, upper]` is bounded below on `[lower, upper)` by its value at `lower`.
theorem monotone_on_right_open_interval_lower_bound_from_closed_interval[Domain: LinearOrder, Value: PartialOrder](
    lower: Domain, upper: Domain, f: Domain -> Value
) {
    lower <= upper and is_monotone_on(closed_interval_set(lower, upper), f)
    implies is_lower_bound_on(right_open_interval_set(lower, upper), f, f(lower))
}

/// A monotone map on `[lower, upper]` is bounded above on `[lower, upper)`.
theorem monotone_on_right_open_interval_bounded_above_from_closed_interval[Domain: LinearOrder, Value: PartialOrder](
    lower: Domain, upper: Domain, f: Domain -> Value
) {
    lower <= upper and is_monotone_on(closed_interval_set(lower, upper), f)
    implies is_bounded_above_on(right_open_interval_set(lower, upper), f)
}

/// A monotone map on `[lower, upper]` is bounded below on `[lower, upper)`.
theorem monotone_on_right_open_interval_bounded_below_from_closed_interval[Domain: LinearOrder, Value: PartialOrder](
    lower: Domain, upper: Domain, f: Domain -> Value
) {
    lower <= upper and is_monotone_on(closed_interval_set(lower, upper), f)
    implies is_bounded_below_on(right_open_interval_set(lower, upper), f)
}

/// A monotone map on `[lower, upper]` is bounded on `[lower, upper)`.
theorem monotone_on_right_open_interval_bounded_from_closed_interval[Domain: LinearOrder, Value: PartialOrder](
    lower: Domain, upper: Domain, f: Domain -> Value
) {
    lower <= upper and is_monotone_on(closed_interval_set(lower, upper), f)
    implies is_bounded_on(right_open_interval_set(lower, upper), f)
}

/// An antitone map on `[lower, upper]` is bounded above by its value at `lower`.
theorem antitone_on_closed_interval_upper_bound[Domain: LinearOrder, Value: PartialOrder](
    lower: Domain, upper: Domain, f: Domain -> Value
) {
    lower <= upper and is_antitone_on(closed_interval_set(lower, upper), f)
    implies is_upper_bound_on(closed_interval_set(lower, upper), f, f(lower))
}

/// An antitone map on `[lower, upper]` is bounded below by its value at `upper`.
theorem antitone_on_closed_interval_lower_bound[Domain: LinearOrder, Value: PartialOrder](
    lower: Domain, upper: Domain, f: Domain -> Value
) {
    lower <= upper and is_antitone_on(closed_interval_set(lower, upper), f)
    implies is_lower_bound_on(closed_interval_set(lower, upper), f, f(upper))
}

/// An antitone map on `[lower, upper]` is bounded above.
theorem antitone_on_closed_interval_bounded_above[Domain: LinearOrder, Value: PartialOrder](
    lower: Domain, upper: Domain, f: Domain -> Value
) {
    lower <= upper and is_antitone_on(closed_interval_set(lower, upper), f)
    implies is_bounded_above_on(closed_interval_set(lower, upper), f)
}

/// An antitone map on `[lower, upper]` is bounded below.
theorem antitone_on_closed_interval_bounded_below[Domain: LinearOrder, Value: PartialOrder](
    lower: Domain, upper: Domain, f: Domain -> Value
) {
    lower <= upper and is_antitone_on(closed_interval_set(lower, upper), f)
    implies is_bounded_below_on(closed_interval_set(lower, upper), f)
}

/// An antitone map on `[lower, upper]` is bounded.
theorem antitone_on_closed_interval_bounded[Domain: LinearOrder, Value: PartialOrder](
    lower: Domain, upper: Domain, f: Domain -> Value
) {
    lower <= upper and is_antitone_on(closed_interval_set(lower, upper), f)
    implies is_bounded_on(closed_interval_set(lower, upper), f)
}

/// An antitone map on `[lower, upper]` is bounded above on `(lower, upper)` by its value at `lower`.
theorem antitone_on_open_interval_upper_bound_from_closed_interval[Domain: LinearOrder, Value: PartialOrder](
    lower: Domain, upper: Domain, f: Domain -> Value
) {
    lower <= upper and is_antitone_on(closed_interval_set(lower, upper), f)
    implies is_upper_bound_on(open_interval_set(lower, upper), f, f(lower))
}

/// An antitone map on `[lower, upper]` is bounded below on `(lower, upper)` by its value at `upper`.
theorem antitone_on_open_interval_lower_bound_from_closed_interval[Domain: LinearOrder, Value: PartialOrder](
    lower: Domain, upper: Domain, f: Domain -> Value
) {
    lower <= upper and is_antitone_on(closed_interval_set(lower, upper), f)
    implies is_lower_bound_on(open_interval_set(lower, upper), f, f(upper))
}

/// An antitone map on `[lower, upper]` is bounded above on `(lower, upper)`.
theorem antitone_on_open_interval_bounded_above_from_closed_interval[Domain: LinearOrder, Value: PartialOrder](
    lower: Domain, upper: Domain, f: Domain -> Value
) {
    lower <= upper and is_antitone_on(closed_interval_set(lower, upper), f)
    implies is_bounded_above_on(open_interval_set(lower, upper), f)
}

/// An antitone map on `[lower, upper]` is bounded below on `(lower, upper)`.
theorem antitone_on_open_interval_bounded_below_from_closed_interval[Domain: LinearOrder, Value: PartialOrder](
    lower: Domain, upper: Domain, f: Domain -> Value
) {
    lower <= upper and is_antitone_on(closed_interval_set(lower, upper), f)
    implies is_bounded_below_on(open_interval_set(lower, upper), f)
}

/// An antitone map on `[lower, upper]` is bounded on `(lower, upper)`.
theorem antitone_on_open_interval_bounded_from_closed_interval[Domain: LinearOrder, Value: PartialOrder](
    lower: Domain, upper: Domain, f: Domain -> Value
) {
    lower <= upper and is_antitone_on(closed_interval_set(lower, upper), f)
    implies is_bounded_on(open_interval_set(lower, upper), f)
}

/// An antitone map on `[lower, upper]` is bounded above on `(lower, upper]` by its value at `lower`.
theorem antitone_on_left_open_interval_upper_bound_from_closed_interval[Domain: LinearOrder, Value: PartialOrder](
    lower: Domain, upper: Domain, f: Domain -> Value
) {
    lower <= upper and is_antitone_on(closed_interval_set(lower, upper), f)
    implies is_upper_bound_on(left_open_interval_set(lower, upper), f, f(lower))
}

/// An antitone map on `[lower, upper]` is bounded below on `(lower, upper]` by its value at `upper`.
theorem antitone_on_left_open_interval_lower_bound_from_closed_interval[Domain: LinearOrder, Value: PartialOrder](
    lower: Domain, upper: Domain, f: Domain -> Value
) {
    lower <= upper and is_antitone_on(closed_interval_set(lower, upper), f)
    implies is_lower_bound_on(left_open_interval_set(lower, upper), f, f(upper))
}

/// An antitone map on `[lower, upper]` is bounded above on `(lower, upper]`.
theorem antitone_on_left_open_interval_bounded_above_from_closed_interval[Domain: LinearOrder, Value: PartialOrder](
    lower: Domain, upper: Domain, f: Domain -> Value
) {
    lower <= upper and is_antitone_on(closed_interval_set(lower, upper), f)
    implies is_bounded_above_on(left_open_interval_set(lower, upper), f)
}

/// An antitone map on `[lower, upper]` is bounded below on `(lower, upper]`.
theorem antitone_on_left_open_interval_bounded_below_from_closed_interval[Domain: LinearOrder, Value: PartialOrder](
    lower: Domain, upper: Domain, f: Domain -> Value
) {
    lower <= upper and is_antitone_on(closed_interval_set(lower, upper), f)
    implies is_bounded_below_on(left_open_interval_set(lower, upper), f)
}

/// An antitone map on `[lower, upper]` is bounded on `(lower, upper]`.
theorem antitone_on_left_open_interval_bounded_from_closed_interval[Domain: LinearOrder, Value: PartialOrder](
    lower: Domain, upper: Domain, f: Domain -> Value
) {
    lower <= upper and is_antitone_on(closed_interval_set(lower, upper), f)
    implies is_bounded_on(left_open_interval_set(lower, upper), f)
}

/// An antitone map on `[lower, upper]` is bounded above on `[lower, upper)` by its value at `lower`.
theorem antitone_on_right_open_interval_upper_bound_from_closed_interval[Domain: LinearOrder, Value: PartialOrder](
    lower: Domain, upper: Domain, f: Domain -> Value
) {
    lower <= upper and is_antitone_on(closed_interval_set(lower, upper), f)
    implies is_upper_bound_on(right_open_interval_set(lower, upper), f, f(lower))
}

/// An antitone map on `[lower, upper]` is bounded below on `[lower, upper)` by its value at `upper`.
theorem antitone_on_right_open_interval_lower_bound_from_closed_interval[Domain: LinearOrder, Value: PartialOrder](
    lower: Domain, upper: Domain, f: Domain -> Value
) {
    lower <= upper and is_antitone_on(closed_interval_set(lower, upper), f)
    implies is_lower_bound_on(right_open_interval_set(lower, upper), f, f(upper))
}

/// An antitone map on `[lower, upper]` is bounded above on `[lower, upper)`.
theorem antitone_on_right_open_interval_bounded_above_from_closed_interval[Domain: LinearOrder, Value: PartialOrder](
    lower: Domain, upper: Domain, f: Domain -> Value
) {
    lower <= upper and is_antitone_on(closed_interval_set(lower, upper), f)
    implies is_bounded_above_on(right_open_interval_set(lower, upper), f)
}

/// An antitone map on `[lower, upper]` is bounded below on `[lower, upper)`.
theorem antitone_on_right_open_interval_bounded_below_from_closed_interval[Domain: LinearOrder, Value: PartialOrder](
    lower: Domain, upper: Domain, f: Domain -> Value
) {
    lower <= upper and is_antitone_on(closed_interval_set(lower, upper), f)
    implies is_bounded_below_on(right_open_interval_set(lower, upper), f)
}

/// An antitone map on `[lower, upper]` is bounded on `[lower, upper)`.
theorem antitone_on_right_open_interval_bounded_from_closed_interval[Domain: LinearOrder, Value: PartialOrder](
    lower: Domain, upper: Domain, f: Domain -> Value
) {
    lower <= upper and is_antitone_on(closed_interval_set(lower, upper), f)
    implies is_bounded_on(right_open_interval_set(lower, upper), f)
}

/// A monotone map on `[lower, upper]` attains its maximum at `upper`.
theorem monotone_on_closed_interval_attains_maximum_at_upper[Domain: LinearOrder, Value: PartialOrder](
    lower: Domain, upper: Domain, f: Domain -> Value
) {
    lower <= upper and is_monotone_on(closed_interval_set(lower, upper), f)
    implies attains_maximum_at(closed_interval_set(lower, upper), f, upper)
}

/// A monotone map on `[lower, upper]` attains its minimum at `lower`.
theorem monotone_on_closed_interval_attains_minimum_at_lower[Domain: LinearOrder, Value: PartialOrder](
    lower: Domain, upper: Domain, f: Domain -> Value
) {
    lower <= upper and is_monotone_on(closed_interval_set(lower, upper), f)
    implies attains_minimum_at(closed_interval_set(lower, upper), f, lower)
}

/// A monotone map on `[lower, upper]` attains a maximum on the interval.
theorem monotone_on_closed_interval_attains_maximum[Domain: LinearOrder, Value: PartialOrder](
    lower: Domain, upper: Domain, f: Domain -> Value
) {
    lower <= upper and is_monotone_on(closed_interval_set(lower, upper), f)
    implies attains_maximum_on(closed_interval_set(lower, upper), f)
}

/// A monotone map on `[lower, upper]` attains a minimum on the interval.
theorem monotone_on_closed_interval_attains_minimum[Domain: LinearOrder, Value: PartialOrder](
    lower: Domain, upper: Domain, f: Domain -> Value
) {
    lower <= upper and is_monotone_on(closed_interval_set(lower, upper), f)
    implies attains_minimum_on(closed_interval_set(lower, upper), f)
}

/// An antitone map on `[lower, upper]` attains its maximum at `lower`.
theorem antitone_on_closed_interval_attains_maximum_at_lower[Domain: LinearOrder, Value: PartialOrder](
    lower: Domain, upper: Domain, f: Domain -> Value
) {
    lower <= upper and is_antitone_on(closed_interval_set(lower, upper), f)
    implies attains_maximum_at(closed_interval_set(lower, upper), f, lower)
}

/// An antitone map on `[lower, upper]` attains its minimum at `upper`.
theorem antitone_on_closed_interval_attains_minimum_at_upper[Domain: LinearOrder, Value: PartialOrder](
    lower: Domain, upper: Domain, f: Domain -> Value
) {
    lower <= upper and is_antitone_on(closed_interval_set(lower, upper), f)
    implies attains_minimum_at(closed_interval_set(lower, upper), f, upper)
}

/// An antitone map on `[lower, upper]` attains a maximum on the interval.
theorem antitone_on_closed_interval_attains_maximum[Domain: LinearOrder, Value: PartialOrder](
    lower: Domain, upper: Domain, f: Domain -> Value
) {
    lower <= upper and is_antitone_on(closed_interval_set(lower, upper), f)
    implies attains_maximum_on(closed_interval_set(lower, upper), f)
}

/// An antitone map on `[lower, upper]` attains a minimum on the interval.
theorem antitone_on_closed_interval_attains_minimum[Domain: LinearOrder, Value: PartialOrder](
    lower: Domain, upper: Domain, f: Domain -> Value
) {
    lower <= upper and is_antitone_on(closed_interval_set(lower, upper), f)
    implies attains_minimum_on(closed_interval_set(lower, upper), f)
}

/// A monotone map on `(lower, upper]` is bounded above by its value at `upper`.
theorem monotone_on_left_open_interval_upper_bound[Domain: LinearOrder, Value: PartialOrder](
    lower: Domain, upper: Domain, f: Domain -> Value
) {
    lower < upper and is_monotone_on(left_open_interval_set(lower, upper), f)
    implies is_upper_bound_on(left_open_interval_set(lower, upper), f, f(upper))
}

/// A monotone map on `(lower, upper]` attains its maximum at `upper`.
theorem monotone_on_left_open_interval_attains_maximum_at_upper[Domain: LinearOrder, Value: PartialOrder](
    lower: Domain, upper: Domain, f: Domain -> Value
) {
    lower < upper and is_monotone_on(left_open_interval_set(lower, upper), f)
    implies attains_maximum_at(left_open_interval_set(lower, upper), f, upper)
}

/// A monotone map on `(lower, upper]` attains a maximum on the interval.
theorem monotone_on_left_open_interval_attains_maximum[Domain: LinearOrder, Value: PartialOrder](
    lower: Domain, upper: Domain, f: Domain -> Value
) {
    lower < upper and is_monotone_on(left_open_interval_set(lower, upper), f)
    implies attains_maximum_on(left_open_interval_set(lower, upper), f)
}

/// A monotone map on `(lower, upper]` is bounded above.
theorem monotone_on_left_open_interval_bounded_above[Domain: LinearOrder, Value: PartialOrder](
    lower: Domain, upper: Domain, f: Domain -> Value
) {
    lower < upper and is_monotone_on(left_open_interval_set(lower, upper), f)
    implies is_bounded_above_on(left_open_interval_set(lower, upper), f)
}

/// A monotone map on `[lower, upper)` is bounded below by its value at `lower`.
theorem monotone_on_right_open_interval_lower_bound[Domain: LinearOrder, Value: PartialOrder](
    lower: Domain, upper: Domain, f: Domain -> Value
) {
    lower < upper and is_monotone_on(right_open_interval_set(lower, upper), f)
    implies is_lower_bound_on(right_open_interval_set(lower, upper), f, f(lower))
}

/// A monotone map on `[lower, upper)` attains its minimum at `lower`.
theorem monotone_on_right_open_interval_attains_minimum_at_lower[Domain: LinearOrder, Value: PartialOrder](
    lower: Domain, upper: Domain, f: Domain -> Value
) {
    lower < upper and is_monotone_on(right_open_interval_set(lower, upper), f)
    implies attains_minimum_at(right_open_interval_set(lower, upper), f, lower)
}

/// A monotone map on `[lower, upper)` attains a minimum on the interval.
theorem monotone_on_right_open_interval_attains_minimum[Domain: LinearOrder, Value: PartialOrder](
    lower: Domain, upper: Domain, f: Domain -> Value
) {
    lower < upper and is_monotone_on(right_open_interval_set(lower, upper), f)
    implies attains_minimum_on(right_open_interval_set(lower, upper), f)
}

/// A monotone map on `[lower, upper)` is bounded below.
theorem monotone_on_right_open_interval_bounded_below[Domain: LinearOrder, Value: PartialOrder](
    lower: Domain, upper: Domain, f: Domain -> Value
) {
    lower < upper and is_monotone_on(right_open_interval_set(lower, upper), f)
    implies is_bounded_below_on(right_open_interval_set(lower, upper), f)
}

/// An antitone map on `(lower, upper]` is bounded below by its value at `upper`.
theorem antitone_on_left_open_interval_lower_bound[Domain: LinearOrder, Value: PartialOrder](
    lower: Domain, upper: Domain, f: Domain -> Value
) {
    lower < upper and is_antitone_on(left_open_interval_set(lower, upper), f)
    implies is_lower_bound_on(left_open_interval_set(lower, upper), f, f(upper))
}

/// An antitone map on `(lower, upper]` attains its minimum at `upper`.
theorem antitone_on_left_open_interval_attains_minimum_at_upper[Domain: LinearOrder, Value: PartialOrder](
    lower: Domain, upper: Domain, f: Domain -> Value
) {
    lower < upper and is_antitone_on(left_open_interval_set(lower, upper), f)
    implies attains_minimum_at(left_open_interval_set(lower, upper), f, upper)
}

/// An antitone map on `(lower, upper]` attains a minimum on the interval.
theorem antitone_on_left_open_interval_attains_minimum[Domain: LinearOrder, Value: PartialOrder](
    lower: Domain, upper: Domain, f: Domain -> Value
) {
    lower < upper and is_antitone_on(left_open_interval_set(lower, upper), f)
    implies attains_minimum_on(left_open_interval_set(lower, upper), f)
}

/// An antitone map on `(lower, upper]` is bounded below.
theorem antitone_on_left_open_interval_bounded_below[Domain: LinearOrder, Value: PartialOrder](
    lower: Domain, upper: Domain, f: Domain -> Value
) {
    lower < upper and is_antitone_on(left_open_interval_set(lower, upper), f)
    implies is_bounded_below_on(left_open_interval_set(lower, upper), f)
}

/// An antitone map on `[lower, upper)` is bounded above by its value at `lower`.
theorem antitone_on_right_open_interval_upper_bound[Domain: LinearOrder, Value: PartialOrder](
    lower: Domain, upper: Domain, f: Domain -> Value
) {
    lower < upper and is_antitone_on(right_open_interval_set(lower, upper), f)
    implies is_upper_bound_on(right_open_interval_set(lower, upper), f, f(lower))
}

/// An antitone map on `[lower, upper)` attains its maximum at `lower`.
theorem antitone_on_right_open_interval_attains_maximum_at_lower[Domain: LinearOrder, Value: PartialOrder](
    lower: Domain, upper: Domain, f: Domain -> Value
) {
    lower < upper and is_antitone_on(right_open_interval_set(lower, upper), f)
    implies attains_maximum_at(right_open_interval_set(lower, upper), f, lower)
}

/// An antitone map on `[lower, upper)` attains a maximum on the interval.
theorem antitone_on_right_open_interval_attains_maximum[Domain: LinearOrder, Value: PartialOrder](
    lower: Domain, upper: Domain, f: Domain -> Value
) {
    lower < upper and is_antitone_on(right_open_interval_set(lower, upper), f)
    implies attains_maximum_on(right_open_interval_set(lower, upper), f)
}

/// An antitone map on `[lower, upper)` is bounded above.
theorem antitone_on_right_open_interval_bounded_above[Domain: LinearOrder, Value: PartialOrder](
    lower: Domain, upper: Domain, f: Domain -> Value
) {
    lower < upper and is_antitone_on(right_open_interval_set(lower, upper), f)
    implies is_bounded_above_on(right_open_interval_set(lower, upper), f)
}
