/// Bounds transported along pointwise function comparisons on a set.

from order import PartialOrder, lte_trans
from order_set.function_order_on import function_lte_on, function_lt_on, function_between_on,
    function_strict_between_on, function_lte_on_apply, function_eq_on, function_eq_on_apply,
    function_eq_on_symm, function_between_on_left, function_between_on_right,
    function_between_on_of_strict_between_on, function_lt_on_imp_lte_on
from order_set.function_bounds import is_upper_bound_on, is_lower_bound_on, is_bounded_above_on,
    is_bounded_below_on, is_bounded_on, attains_maximum_at, attains_minimum_at,
    attains_maximum_on, attains_minimum_on, upper_bound_on_apply, lower_bound_on_apply,
    maximum_value_is_upper_bound_on, minimum_value_is_lower_bound_on
from data.basic.set import Set

/// An upper bound for a larger function is an upper bound for any pointwise smaller function.
theorem upper_bound_on_of_function_lte_on[Domain, Value: PartialOrder](
    domain: Set[Domain], lower: Domain -> Value, upper: Domain -> Value, bound: Value
) {
    function_lte_on(domain, lower, upper) and is_upper_bound_on(domain, upper, bound)
    implies is_upper_bound_on(domain, lower, bound)
} by {
    forall(point: Domain) {
        if domain.contains(point) {
            function_lte_on_apply(domain, lower, upper, point)
            upper_bound_on_apply(domain, upper, bound, point)
            lower(point) <= upper(point)
            upper(point) <= bound
            lte_trans(lower(point), upper(point), bound)
            lower(point) <= bound
        }
    }
}

/// A lower bound for a smaller function is a lower bound for any pointwise larger function.
theorem lower_bound_on_of_function_lte_on[Domain, Value: PartialOrder](
    domain: Set[Domain], lower: Domain -> Value, upper: Domain -> Value, bound: Value
) {
    function_lte_on(domain, lower, upper) and is_lower_bound_on(domain, lower, bound)
    implies is_lower_bound_on(domain, upper, bound)
} by {
    forall(point: Domain) {
        if domain.contains(point) {
            lower_bound_on_apply(domain, lower, bound, point)
            function_lte_on_apply(domain, lower, upper, point)
            bound <= lower(point)
            lower(point) <= upper(point)
            lte_trans(bound, lower(point), upper(point))
            bound <= upper(point)
        }
    }
}

/// Boundedness above descends along a pointwise comparison.
theorem bounded_above_on_of_function_lte_on[Domain, Value: PartialOrder](
    domain: Set[Domain], lower: Domain -> Value, upper: Domain -> Value
) {
    function_lte_on(domain, lower, upper) and is_bounded_above_on(domain, upper)
    implies is_bounded_above_on(domain, lower)
} by {
    let bound: Value satisfy {
        is_upper_bound_on(domain, upper, bound)
    }
    upper_bound_on_of_function_lte_on(domain, lower, upper, bound)
    is_upper_bound_on(domain, lower, bound)
    exists(witness: Value) {
        is_upper_bound_on(domain, lower, witness)
    }
    is_bounded_above_on(domain, lower)
}

/// Boundedness below ascends along a pointwise comparison.
theorem bounded_below_on_of_function_lte_on[Domain, Value: PartialOrder](
    domain: Set[Domain], lower: Domain -> Value, upper: Domain -> Value
) {
    function_lte_on(domain, lower, upper) and is_bounded_below_on(domain, lower)
    implies is_bounded_below_on(domain, upper)
} by {
    let bound: Value satisfy {
        is_lower_bound_on(domain, lower, bound)
    }
    lower_bound_on_of_function_lte_on(domain, lower, upper, bound)
    is_lower_bound_on(domain, upper, bound)
    exists(witness: Value) {
        is_lower_bound_on(domain, upper, witness)
    }
    is_bounded_below_on(domain, upper)
}

/// An upper bound descends along a strict pointwise comparison.
theorem upper_bound_on_of_function_lt_on[Domain, Value: PartialOrder](
    domain: Set[Domain], lower: Domain -> Value, upper: Domain -> Value, bound: Value
) {
    function_lt_on(domain, lower, upper) and is_upper_bound_on(domain, upper, bound)
    implies is_upper_bound_on(domain, lower, bound)
} by {
    function_lt_on_imp_lte_on(domain, lower, upper)
    function_lte_on(domain, lower, upper)
    upper_bound_on_of_function_lte_on(domain, lower, upper, bound)
    is_upper_bound_on(domain, lower, bound)
}

/// A lower bound ascends along a strict pointwise comparison.
theorem lower_bound_on_of_function_lt_on[Domain, Value: PartialOrder](
    domain: Set[Domain], lower: Domain -> Value, upper: Domain -> Value, bound: Value
) {
    function_lt_on(domain, lower, upper) and is_lower_bound_on(domain, lower, bound)
    implies is_lower_bound_on(domain, upper, bound)
} by {
    function_lt_on_imp_lte_on(domain, lower, upper)
    function_lte_on(domain, lower, upper)
    lower_bound_on_of_function_lte_on(domain, lower, upper, bound)
    is_lower_bound_on(domain, upper, bound)
}

/// Boundedness above descends along a strict pointwise comparison.
theorem bounded_above_on_of_function_lt_on[Domain, Value: PartialOrder](
    domain: Set[Domain], lower: Domain -> Value, upper: Domain -> Value
) {
    function_lt_on(domain, lower, upper) and is_bounded_above_on(domain, upper)
    implies is_bounded_above_on(domain, lower)
} by {
    function_lt_on_imp_lte_on(domain, lower, upper)
    function_lte_on(domain, lower, upper)
    bounded_above_on_of_function_lte_on(domain, lower, upper)
    is_bounded_above_on(domain, lower)
}

/// Boundedness below ascends along a strict pointwise comparison.
theorem bounded_below_on_of_function_lt_on[Domain, Value: PartialOrder](
    domain: Set[Domain], lower: Domain -> Value, upper: Domain -> Value
) {
    function_lt_on(domain, lower, upper) and is_bounded_below_on(domain, lower)
    implies is_bounded_below_on(domain, upper)
} by {
    function_lt_on_imp_lte_on(domain, lower, upper)
    function_lte_on(domain, lower, upper)
    bounded_below_on_of_function_lte_on(domain, lower, upper)
    is_bounded_below_on(domain, upper)
}

/// Both sides of a pointwise comparison are bounded above when the larger side is bounded above.
theorem bounded_above_on_pair_of_function_lte_on[Domain, Value: PartialOrder](
    domain: Set[Domain], lower: Domain -> Value, upper: Domain -> Value
) {
    function_lte_on(domain, lower, upper) and is_bounded_above_on(domain, upper)
    implies is_bounded_above_on(domain, lower) and is_bounded_above_on(domain, upper)
} by {
    bounded_above_on_of_function_lte_on(domain, lower, upper)
    is_bounded_above_on(domain, lower)
    is_bounded_above_on(domain, upper)
}

/// Both sides of a pointwise comparison are bounded below when the smaller side is bounded below.
theorem bounded_below_on_pair_of_function_lte_on[Domain, Value: PartialOrder](
    domain: Set[Domain], lower: Domain -> Value, upper: Domain -> Value
) {
    function_lte_on(domain, lower, upper) and is_bounded_below_on(domain, lower)
    implies is_bounded_below_on(domain, lower) and is_bounded_below_on(domain, upper)
} by {
    bounded_below_on_of_function_lte_on(domain, lower, upper)
    is_bounded_below_on(domain, lower)
    is_bounded_below_on(domain, upper)
}

/// The middle of a pointwise sandwich is bounded above when the upper function is bounded above.
theorem bounded_above_on_middle_of_function_lte_on[Domain, Value: PartialOrder](
    domain: Set[Domain], lower: Domain -> Value, middle: Domain -> Value, upper: Domain -> Value
) {
    function_lte_on(domain, lower, middle) and function_lte_on(domain, middle, upper) and
    is_bounded_above_on(domain, upper)
    implies is_bounded_above_on(domain, middle)
} by {
    bounded_above_on_of_function_lte_on(domain, middle, upper)
    is_bounded_above_on(domain, middle)
}

/// The middle of a pointwise sandwich is bounded below when the lower function is bounded below.
theorem bounded_below_on_middle_of_function_lte_on[Domain, Value: PartialOrder](
    domain: Set[Domain], lower: Domain -> Value, middle: Domain -> Value, upper: Domain -> Value
) {
    function_lte_on(domain, lower, middle) and function_lte_on(domain, middle, upper) and
    is_bounded_below_on(domain, lower)
    implies is_bounded_below_on(domain, middle)
} by {
    bounded_below_on_of_function_lte_on(domain, lower, middle)
    is_bounded_below_on(domain, middle)
}

/// Both sides of a strict pointwise comparison are bounded above when the larger side is bounded above.
theorem bounded_above_on_pair_of_function_lt_on[Domain, Value: PartialOrder](
    domain: Set[Domain], lower: Domain -> Value, upper: Domain -> Value
) {
    function_lt_on(domain, lower, upper) and is_bounded_above_on(domain, upper)
    implies is_bounded_above_on(domain, lower) and is_bounded_above_on(domain, upper)
} by {
    function_lt_on_imp_lte_on(domain, lower, upper)
    bounded_above_on_pair_of_function_lte_on(domain, lower, upper)
    is_bounded_above_on(domain, lower)
    is_bounded_above_on(domain, upper)
}

/// Both sides of a strict pointwise comparison are bounded below when the smaller side is bounded below.
theorem bounded_below_on_pair_of_function_lt_on[Domain, Value: PartialOrder](
    domain: Set[Domain], lower: Domain -> Value, upper: Domain -> Value
) {
    function_lt_on(domain, lower, upper) and is_bounded_below_on(domain, lower)
    implies is_bounded_below_on(domain, lower) and is_bounded_below_on(domain, upper)
} by {
    function_lt_on_imp_lte_on(domain, lower, upper)
    bounded_below_on_pair_of_function_lte_on(domain, lower, upper)
    is_bounded_below_on(domain, lower)
    is_bounded_below_on(domain, upper)
}

/// The middle of a strict pointwise sandwich is bounded above when the upper function is bounded above.
theorem bounded_above_on_middle_of_function_lt_on[Domain, Value: PartialOrder](
    domain: Set[Domain], lower: Domain -> Value, middle: Domain -> Value, upper: Domain -> Value
) {
    function_lt_on(domain, lower, middle) and function_lt_on(domain, middle, upper) and
    is_bounded_above_on(domain, upper)
    implies is_bounded_above_on(domain, middle)
} by {
    function_lt_on_imp_lte_on(domain, lower, middle)
    function_lt_on_imp_lte_on(domain, middle, upper)
    bounded_above_on_middle_of_function_lte_on(domain, lower, middle, upper)
    is_bounded_above_on(domain, middle)
}

/// The middle of a strict pointwise sandwich is bounded below when the lower function is bounded below.
theorem bounded_below_on_middle_of_function_lt_on[Domain, Value: PartialOrder](
    domain: Set[Domain], lower: Domain -> Value, middle: Domain -> Value, upper: Domain -> Value
) {
    function_lt_on(domain, lower, middle) and function_lt_on(domain, middle, upper) and
    is_bounded_below_on(domain, lower)
    implies is_bounded_below_on(domain, middle)
} by {
    function_lt_on_imp_lte_on(domain, lower, middle)
    function_lt_on_imp_lte_on(domain, middle, upper)
    bounded_below_on_middle_of_function_lte_on(domain, lower, middle, upper)
    is_bounded_below_on(domain, middle)
}

/// A pointwise smaller function is bounded when it has a lower bound and the larger function has an upper bound.
theorem bounded_on_lower_of_function_lte_on[Domain, Value: PartialOrder](
    domain: Set[Domain], lower: Domain -> Value, upper: Domain -> Value
) {
    function_lte_on(domain, lower, upper) and is_bounded_below_on(domain, lower) and
    is_bounded_above_on(domain, upper)
    implies is_bounded_on(domain, lower)
} by {
    bounded_above_on_of_function_lte_on(domain, lower, upper)
    is_bounded_above_on(domain, lower)
    is_bounded_below_on(domain, lower)
    is_bounded_on(domain, lower)
}

/// A pointwise larger function is bounded when the smaller function has a lower bound and it has an upper bound.
theorem bounded_on_upper_of_function_lte_on[Domain, Value: PartialOrder](
    domain: Set[Domain], lower: Domain -> Value, upper: Domain -> Value
) {
    function_lte_on(domain, lower, upper) and is_bounded_below_on(domain, lower) and
    is_bounded_above_on(domain, upper)
    implies is_bounded_on(domain, upper)
} by {
    bounded_below_on_of_function_lte_on(domain, lower, upper)
    is_bounded_below_on(domain, upper)
    is_bounded_above_on(domain, upper)
    is_bounded_on(domain, upper)
}

/// Both sides of a pointwise comparison are bounded when the lower side has a lower bound and the upper side has an upper bound.
theorem bounded_on_pair_of_function_lte_on[Domain, Value: PartialOrder](
    domain: Set[Domain], lower: Domain -> Value, upper: Domain -> Value
) {
    function_lte_on(domain, lower, upper) and is_bounded_below_on(domain, lower) and
    is_bounded_above_on(domain, upper)
    implies is_bounded_on(domain, lower) and is_bounded_on(domain, upper)
} by {
    bounded_on_lower_of_function_lte_on(domain, lower, upper)
    bounded_on_upper_of_function_lte_on(domain, lower, upper)
    is_bounded_on(domain, lower)
    is_bounded_on(domain, upper)
}

/// A function between lower and upper pointwise bounds is bounded on the same set.
theorem bounded_on_middle_of_function_lte_on[Domain, Value: PartialOrder](
    domain: Set[Domain], lower: Domain -> Value, middle: Domain -> Value, upper: Domain -> Value
) {
    function_lte_on(domain, lower, middle) and function_lte_on(domain, middle, upper) and
    is_bounded_below_on(domain, lower) and is_bounded_above_on(domain, upper)
    implies is_bounded_on(domain, middle)
} by {
    bounded_below_on_of_function_lte_on(domain, lower, middle)
    bounded_above_on_of_function_lte_on(domain, middle, upper)
    is_bounded_below_on(domain, middle)
    is_bounded_above_on(domain, middle)
    is_bounded_on(domain, middle)
}

/// A strictly smaller function is bounded under the corresponding endpoint bounds.
theorem bounded_on_lower_of_function_lt_on[Domain, Value: PartialOrder](
    domain: Set[Domain], lower: Domain -> Value, upper: Domain -> Value
) {
    function_lt_on(domain, lower, upper) and is_bounded_below_on(domain, lower) and
    is_bounded_above_on(domain, upper)
    implies is_bounded_on(domain, lower)
} by {
    function_lt_on_imp_lte_on(domain, lower, upper)
    bounded_on_lower_of_function_lte_on(domain, lower, upper)
    is_bounded_on(domain, lower)
}

/// A strictly larger function is bounded under the corresponding endpoint bounds.
theorem bounded_on_upper_of_function_lt_on[Domain, Value: PartialOrder](
    domain: Set[Domain], lower: Domain -> Value, upper: Domain -> Value
) {
    function_lt_on(domain, lower, upper) and is_bounded_below_on(domain, lower) and
    is_bounded_above_on(domain, upper)
    implies is_bounded_on(domain, upper)
} by {
    function_lt_on_imp_lte_on(domain, lower, upper)
    bounded_on_upper_of_function_lte_on(domain, lower, upper)
    is_bounded_on(domain, upper)
}

/// Both sides of a strict pointwise comparison are bounded under the corresponding endpoint bounds.
theorem bounded_on_pair_of_function_lt_on[Domain, Value: PartialOrder](
    domain: Set[Domain], lower: Domain -> Value, upper: Domain -> Value
) {
    function_lt_on(domain, lower, upper) and is_bounded_below_on(domain, lower) and
    is_bounded_above_on(domain, upper)
    implies is_bounded_on(domain, lower) and is_bounded_on(domain, upper)
} by {
    function_lt_on_imp_lte_on(domain, lower, upper)
    bounded_on_pair_of_function_lte_on(domain, lower, upper)
    is_bounded_on(domain, lower)
    is_bounded_on(domain, upper)
}

/// A function between strict pointwise lower and upper bounds is bounded on the same set.
theorem bounded_on_middle_of_function_lt_on[Domain, Value: PartialOrder](
    domain: Set[Domain], lower: Domain -> Value, middle: Domain -> Value, upper: Domain -> Value
) {
    function_lt_on(domain, lower, middle) and function_lt_on(domain, middle, upper) and
    is_bounded_below_on(domain, lower) and is_bounded_above_on(domain, upper)
    implies is_bounded_on(domain, middle)
} by {
    function_lt_on_imp_lte_on(domain, lower, middle)
    function_lt_on_imp_lte_on(domain, middle, upper)
    bounded_on_middle_of_function_lte_on(domain, lower, middle, upper)
    is_bounded_on(domain, middle)
}


/// An upper bound for the upper function bounds the middle function in a pointwise between relation.
theorem upper_bound_on_middle_of_function_between_on[Domain, Value: PartialOrder](
    domain: Set[Domain], lower: Domain -> Value, middle: Domain -> Value,
    upper: Domain -> Value, bound: Value
) {
    function_between_on(domain, lower, middle, upper) and is_upper_bound_on(domain, upper, bound)
    implies is_upper_bound_on(domain, middle, bound)
} by {
    function_between_on_right(domain, lower, middle, upper)
    upper_bound_on_of_function_lte_on(domain, middle, upper, bound)
    is_upper_bound_on(domain, middle, bound)
}

/// A lower bound for the lower function bounds the middle function in a pointwise between relation.
theorem lower_bound_on_middle_of_function_between_on[Domain, Value: PartialOrder](
    domain: Set[Domain], lower: Domain -> Value, middle: Domain -> Value,
    upper: Domain -> Value, bound: Value
) {
    function_between_on(domain, lower, middle, upper) and is_lower_bound_on(domain, lower, bound)
    implies is_lower_bound_on(domain, middle, bound)
} by {
    function_between_on_left(domain, lower, middle, upper)
    lower_bound_on_of_function_lte_on(domain, lower, middle, bound)
    is_lower_bound_on(domain, middle, bound)
}

/// An upper bound for the upper function bounds the lower function in a pointwise between relation.
theorem upper_bound_on_lower_of_function_between_on[Domain, Value: PartialOrder](
    domain: Set[Domain], lower: Domain -> Value, middle: Domain -> Value,
    upper: Domain -> Value, bound: Value
) {
    function_between_on(domain, lower, middle, upper) and is_upper_bound_on(domain, upper, bound)
    implies is_upper_bound_on(domain, lower, bound)
} by {
    function_between_on_left(domain, lower, middle, upper)
    upper_bound_on_middle_of_function_between_on(domain, lower, middle, upper, bound)
    is_upper_bound_on(domain, middle, bound)
    upper_bound_on_of_function_lte_on(domain, lower, middle, bound)
    is_upper_bound_on(domain, lower, bound)
}

/// A lower bound for the lower function bounds the upper function in a pointwise between relation.
theorem lower_bound_on_upper_of_function_between_on[Domain, Value: PartialOrder](
    domain: Set[Domain], lower: Domain -> Value, middle: Domain -> Value,
    upper: Domain -> Value, bound: Value
) {
    function_between_on(domain, lower, middle, upper) and is_lower_bound_on(domain, lower, bound)
    implies is_lower_bound_on(domain, upper, bound)
} by {
    function_between_on_right(domain, lower, middle, upper)
    lower_bound_on_middle_of_function_between_on(domain, lower, middle, upper, bound)
    is_lower_bound_on(domain, middle, bound)
    lower_bound_on_of_function_lte_on(domain, middle, upper, bound)
    is_lower_bound_on(domain, upper, bound)
}

/// An upper bound for the upper function bounds the middle function in a strict pointwise between relation.
theorem upper_bound_on_middle_of_function_strict_between_on[Domain, Value: PartialOrder](
    domain: Set[Domain], lower: Domain -> Value, middle: Domain -> Value,
    upper: Domain -> Value, bound: Value
) {
    function_strict_between_on(domain, lower, middle, upper) and is_upper_bound_on(domain, upper, bound)
    implies is_upper_bound_on(domain, middle, bound)
} by {
    function_between_on_of_strict_between_on(domain, lower, middle, upper)
    upper_bound_on_middle_of_function_between_on(domain, lower, middle, upper, bound)
    is_upper_bound_on(domain, middle, bound)
}

/// A lower bound for the lower function bounds the middle function in a strict pointwise between relation.
theorem lower_bound_on_middle_of_function_strict_between_on[Domain, Value: PartialOrder](
    domain: Set[Domain], lower: Domain -> Value, middle: Domain -> Value,
    upper: Domain -> Value, bound: Value
) {
    function_strict_between_on(domain, lower, middle, upper) and is_lower_bound_on(domain, lower, bound)
    implies is_lower_bound_on(domain, middle, bound)
} by {
    function_between_on_of_strict_between_on(domain, lower, middle, upper)
    lower_bound_on_middle_of_function_between_on(domain, lower, middle, upper, bound)
    is_lower_bound_on(domain, middle, bound)
}

/// An upper bound for the upper function bounds the lower function in a strict pointwise between relation.
theorem upper_bound_on_lower_of_function_strict_between_on[Domain, Value: PartialOrder](
    domain: Set[Domain], lower: Domain -> Value, middle: Domain -> Value,
    upper: Domain -> Value, bound: Value
) {
    function_strict_between_on(domain, lower, middle, upper) and is_upper_bound_on(domain, upper, bound)
    implies is_upper_bound_on(domain, lower, bound)
} by {
    function_between_on_of_strict_between_on(domain, lower, middle, upper)
    upper_bound_on_lower_of_function_between_on(domain, lower, middle, upper, bound)
    is_upper_bound_on(domain, lower, bound)
}

/// A lower bound for the lower function bounds the upper function in a strict pointwise between relation.
theorem lower_bound_on_upper_of_function_strict_between_on[Domain, Value: PartialOrder](
    domain: Set[Domain], lower: Domain -> Value, middle: Domain -> Value,
    upper: Domain -> Value, bound: Value
) {
    function_strict_between_on(domain, lower, middle, upper) and is_lower_bound_on(domain, lower, bound)
    implies is_lower_bound_on(domain, upper, bound)
} by {
    function_between_on_of_strict_between_on(domain, lower, middle, upper)
    lower_bound_on_upper_of_function_between_on(domain, lower, middle, upper, bound)
    is_lower_bound_on(domain, upper, bound)
}

/// The middle of a pointwise between relation is bounded above when the upper function is bounded above.
theorem bounded_above_on_middle_of_function_between_on[Domain, Value: PartialOrder](
    domain: Set[Domain], lower: Domain -> Value, middle: Domain -> Value, upper: Domain -> Value
) {
    function_between_on(domain, lower, middle, upper) and is_bounded_above_on(domain, upper)
    implies is_bounded_above_on(domain, middle)
} by {
    function_between_on_right(domain, lower, middle, upper)
    bounded_above_on_of_function_lte_on(domain, middle, upper)
    is_bounded_above_on(domain, middle)
}

/// The middle of a pointwise between relation is bounded below when the lower function is bounded below.
theorem bounded_below_on_middle_of_function_between_on[Domain, Value: PartialOrder](
    domain: Set[Domain], lower: Domain -> Value, middle: Domain -> Value, upper: Domain -> Value
) {
    function_between_on(domain, lower, middle, upper) and is_bounded_below_on(domain, lower)
    implies is_bounded_below_on(domain, middle)
} by {
    function_between_on_left(domain, lower, middle, upper)
    bounded_below_on_of_function_lte_on(domain, lower, middle)
    is_bounded_below_on(domain, middle)
}

/// The lower function in a pointwise between relation is bounded above when the upper function is bounded above.
theorem bounded_above_on_lower_of_function_between_on[Domain, Value: PartialOrder](
    domain: Set[Domain], lower: Domain -> Value, middle: Domain -> Value, upper: Domain -> Value
) {
    function_between_on(domain, lower, middle, upper) and is_bounded_above_on(domain, upper)
    implies is_bounded_above_on(domain, lower)
} by {
    function_between_on_left(domain, lower, middle, upper)
    bounded_above_on_middle_of_function_between_on(domain, lower, middle, upper)
    is_bounded_above_on(domain, middle)
    bounded_above_on_of_function_lte_on(domain, lower, middle)
    is_bounded_above_on(domain, lower)
}

/// The upper function in a pointwise between relation is bounded below when the lower function is bounded below.
theorem bounded_below_on_upper_of_function_between_on[Domain, Value: PartialOrder](
    domain: Set[Domain], lower: Domain -> Value, middle: Domain -> Value, upper: Domain -> Value
) {
    function_between_on(domain, lower, middle, upper) and is_bounded_below_on(domain, lower)
    implies is_bounded_below_on(domain, upper)
} by {
    function_between_on_right(domain, lower, middle, upper)
    bounded_below_on_middle_of_function_between_on(domain, lower, middle, upper)
    is_bounded_below_on(domain, middle)
    bounded_below_on_of_function_lte_on(domain, middle, upper)
    is_bounded_below_on(domain, upper)
}

/// The middle of a pointwise between relation is bounded under endpoint bounds.
theorem bounded_on_middle_of_function_between_on[Domain, Value: PartialOrder](
    domain: Set[Domain], lower: Domain -> Value, middle: Domain -> Value, upper: Domain -> Value
) {
    function_between_on(domain, lower, middle, upper) and is_bounded_below_on(domain, lower) and
    is_bounded_above_on(domain, upper)
    implies is_bounded_on(domain, middle)
} by {
    bounded_below_on_middle_of_function_between_on(domain, lower, middle, upper)
    bounded_above_on_middle_of_function_between_on(domain, lower, middle, upper)
    is_bounded_below_on(domain, middle)
    is_bounded_above_on(domain, middle)
    is_bounded_on(domain, middle)
}

/// The lower function in a pointwise between relation is bounded under endpoint bounds.
theorem bounded_on_lower_of_function_between_on[Domain, Value: PartialOrder](
    domain: Set[Domain], lower: Domain -> Value, middle: Domain -> Value, upper: Domain -> Value
) {
    function_between_on(domain, lower, middle, upper) and is_bounded_below_on(domain, lower) and
    is_bounded_above_on(domain, upper)
    implies is_bounded_on(domain, lower)
} by {
    bounded_above_on_lower_of_function_between_on(domain, lower, middle, upper)
    is_bounded_above_on(domain, lower)
    is_bounded_below_on(domain, lower)
    is_bounded_on(domain, lower)
}

/// The upper function in a pointwise between relation is bounded under endpoint bounds.
theorem bounded_on_upper_of_function_between_on[Domain, Value: PartialOrder](
    domain: Set[Domain], lower: Domain -> Value, middle: Domain -> Value, upper: Domain -> Value
) {
    function_between_on(domain, lower, middle, upper) and is_bounded_below_on(domain, lower) and
    is_bounded_above_on(domain, upper)
    implies is_bounded_on(domain, upper)
} by {
    bounded_below_on_upper_of_function_between_on(domain, lower, middle, upper)
    is_bounded_below_on(domain, upper)
    is_bounded_above_on(domain, upper)
    is_bounded_on(domain, upper)
}

/// All three functions in a pointwise between relation are bounded under endpoint bounds.
theorem bounded_on_triple_of_function_between_on[Domain, Value: PartialOrder](
    domain: Set[Domain], lower: Domain -> Value, middle: Domain -> Value, upper: Domain -> Value
) {
    function_between_on(domain, lower, middle, upper) and is_bounded_below_on(domain, lower) and
    is_bounded_above_on(domain, upper)
    implies is_bounded_on(domain, lower) and is_bounded_on(domain, middle) and is_bounded_on(domain, upper)
} by {
    bounded_on_lower_of_function_between_on(domain, lower, middle, upper)
    bounded_on_middle_of_function_between_on(domain, lower, middle, upper)
    bounded_on_upper_of_function_between_on(domain, lower, middle, upper)
    is_bounded_on(domain, lower)
    is_bounded_on(domain, middle)
    is_bounded_on(domain, upper)
}

/// The middle of a strict pointwise between relation is bounded above when the upper function is bounded above.
theorem bounded_above_on_middle_of_function_strict_between_on[Domain, Value: PartialOrder](
    domain: Set[Domain], lower: Domain -> Value, middle: Domain -> Value, upper: Domain -> Value
) {
    function_strict_between_on(domain, lower, middle, upper) and is_bounded_above_on(domain, upper)
    implies is_bounded_above_on(domain, middle)
} by {
    function_between_on_of_strict_between_on(domain, lower, middle, upper)
    bounded_above_on_middle_of_function_between_on(domain, lower, middle, upper)
    is_bounded_above_on(domain, middle)
}

/// The middle of a strict pointwise between relation is bounded below when the lower function is bounded below.
theorem bounded_below_on_middle_of_function_strict_between_on[Domain, Value: PartialOrder](
    domain: Set[Domain], lower: Domain -> Value, middle: Domain -> Value, upper: Domain -> Value
) {
    function_strict_between_on(domain, lower, middle, upper) and is_bounded_below_on(domain, lower)
    implies is_bounded_below_on(domain, middle)
} by {
    function_between_on_of_strict_between_on(domain, lower, middle, upper)
    bounded_below_on_middle_of_function_between_on(domain, lower, middle, upper)
    is_bounded_below_on(domain, middle)
}

/// The lower function in a strict pointwise between relation is bounded above when the upper function is bounded above.
theorem bounded_above_on_lower_of_function_strict_between_on[Domain, Value: PartialOrder](
    domain: Set[Domain], lower: Domain -> Value, middle: Domain -> Value, upper: Domain -> Value
) {
    function_strict_between_on(domain, lower, middle, upper) and is_bounded_above_on(domain, upper)
    implies is_bounded_above_on(domain, lower)
} by {
    function_between_on_of_strict_between_on(domain, lower, middle, upper)
    bounded_above_on_lower_of_function_between_on(domain, lower, middle, upper)
    is_bounded_above_on(domain, lower)
}

/// The upper function in a strict pointwise between relation is bounded below when the lower function is bounded below.
theorem bounded_below_on_upper_of_function_strict_between_on[Domain, Value: PartialOrder](
    domain: Set[Domain], lower: Domain -> Value, middle: Domain -> Value, upper: Domain -> Value
) {
    function_strict_between_on(domain, lower, middle, upper) and is_bounded_below_on(domain, lower)
    implies is_bounded_below_on(domain, upper)
} by {
    function_between_on_of_strict_between_on(domain, lower, middle, upper)
    bounded_below_on_upper_of_function_between_on(domain, lower, middle, upper)
    is_bounded_below_on(domain, upper)
}

/// The middle of a strict pointwise between relation is bounded under endpoint bounds.
theorem bounded_on_middle_of_function_strict_between_on[Domain, Value: PartialOrder](
    domain: Set[Domain], lower: Domain -> Value, middle: Domain -> Value, upper: Domain -> Value
) {
    function_strict_between_on(domain, lower, middle, upper) and is_bounded_below_on(domain, lower) and
    is_bounded_above_on(domain, upper)
    implies is_bounded_on(domain, middle)
} by {
    function_between_on_of_strict_between_on(domain, lower, middle, upper)
    bounded_on_middle_of_function_between_on(domain, lower, middle, upper)
    is_bounded_on(domain, middle)
}

/// The lower function in a strict pointwise between relation is bounded under endpoint bounds.
theorem bounded_on_lower_of_function_strict_between_on[Domain, Value: PartialOrder](
    domain: Set[Domain], lower: Domain -> Value, middle: Domain -> Value, upper: Domain -> Value
) {
    function_strict_between_on(domain, lower, middle, upper) and is_bounded_below_on(domain, lower) and
    is_bounded_above_on(domain, upper)
    implies is_bounded_on(domain, lower)
} by {
    function_between_on_of_strict_between_on(domain, lower, middle, upper)
    bounded_on_lower_of_function_between_on(domain, lower, middle, upper)
    is_bounded_on(domain, lower)
}

/// The upper function in a strict pointwise between relation is bounded under endpoint bounds.
theorem bounded_on_upper_of_function_strict_between_on[Domain, Value: PartialOrder](
    domain: Set[Domain], lower: Domain -> Value, middle: Domain -> Value, upper: Domain -> Value
) {
    function_strict_between_on(domain, lower, middle, upper) and is_bounded_below_on(domain, lower) and
    is_bounded_above_on(domain, upper)
    implies is_bounded_on(domain, upper)
} by {
    function_between_on_of_strict_between_on(domain, lower, middle, upper)
    bounded_on_upper_of_function_between_on(domain, lower, middle, upper)
    is_bounded_on(domain, upper)
}

/// All three functions in a strict pointwise between relation are bounded under endpoint bounds.
theorem bounded_on_triple_of_function_strict_between_on[Domain, Value: PartialOrder](
    domain: Set[Domain], lower: Domain -> Value, middle: Domain -> Value, upper: Domain -> Value
) {
    function_strict_between_on(domain, lower, middle, upper) and is_bounded_below_on(domain, lower) and
    is_bounded_above_on(domain, upper)
    implies is_bounded_on(domain, lower) and is_bounded_on(domain, middle) and is_bounded_on(domain, upper)
} by {
    function_between_on_of_strict_between_on(domain, lower, middle, upper)
    bounded_on_triple_of_function_between_on(domain, lower, middle, upper)
    is_bounded_on(domain, lower)
    is_bounded_on(domain, middle)
    is_bounded_on(domain, upper)
}

/// An upper bound is preserved when the bounded function is replaced by an equal function on the domain.
theorem upper_bound_on_of_function_eq_on[Domain, Value: PartialOrder](
    domain: Set[Domain], original: Domain -> Value, replacement: Domain -> Value, bound: Value
) {
    function_eq_on(domain, original, replacement) and is_upper_bound_on(domain, original, bound)
    implies is_upper_bound_on(domain, replacement, bound)
} by {
    forall(point: Domain) {
        if domain.contains(point) {
            function_eq_on_apply(domain, original, replacement, point)
            upper_bound_on_apply(domain, original, bound, point)
            original(point) = replacement(point)
            original(point) <= bound
            replacement(point) <= bound
        }
    }
}

/// A lower bound is preserved when the bounded function is replaced by an equal function on the domain.
theorem lower_bound_on_of_function_eq_on[Domain, Value: PartialOrder](
    domain: Set[Domain], original: Domain -> Value, replacement: Domain -> Value, bound: Value
) {
    function_eq_on(domain, original, replacement) and is_lower_bound_on(domain, original, bound)
    implies is_lower_bound_on(domain, replacement, bound)
} by {
    forall(point: Domain) {
        if domain.contains(point) {
            function_eq_on_apply(domain, original, replacement, point)
            lower_bound_on_apply(domain, original, bound, point)
            original(point) = replacement(point)
            bound <= original(point)
            bound <= replacement(point)
        }
    }
}

/// An upper bound is reflected through equality on the domain.
theorem upper_bound_on_reflect_function_eq_on[Domain, Value: PartialOrder](
    domain: Set[Domain], original: Domain -> Value, replacement: Domain -> Value, bound: Value
) {
    function_eq_on(domain, original, replacement) and is_upper_bound_on(domain, replacement, bound)
    implies is_upper_bound_on(domain, original, bound)
} by {
    function_eq_on_symm(domain, original, replacement)
    function_eq_on(domain, replacement, original)
    upper_bound_on_of_function_eq_on(domain, replacement, original, bound)
    is_upper_bound_on(domain, original, bound)
}

/// A lower bound is reflected through equality on the domain.
theorem lower_bound_on_reflect_function_eq_on[Domain, Value: PartialOrder](
    domain: Set[Domain], original: Domain -> Value, replacement: Domain -> Value, bound: Value
) {
    function_eq_on(domain, original, replacement) and is_lower_bound_on(domain, replacement, bound)
    implies is_lower_bound_on(domain, original, bound)
} by {
    function_eq_on_symm(domain, original, replacement)
    function_eq_on(domain, replacement, original)
    lower_bound_on_of_function_eq_on(domain, replacement, original, bound)
    is_lower_bound_on(domain, original, bound)
}

/// Boundedness above is preserved when replacing a function by an equal function on the domain.
theorem bounded_above_on_of_function_eq_on[Domain, Value: PartialOrder](
    domain: Set[Domain], original: Domain -> Value, replacement: Domain -> Value
) {
    function_eq_on(domain, original, replacement) and is_bounded_above_on(domain, original)
    implies is_bounded_above_on(domain, replacement)
} by {
    let bound: Value satisfy {
        is_upper_bound_on(domain, original, bound)
    }
    upper_bound_on_of_function_eq_on(domain, original, replacement, bound)
    is_upper_bound_on(domain, replacement, bound)
    exists(witness: Value) {
        is_upper_bound_on(domain, replacement, witness)
    }
    is_bounded_above_on(domain, replacement)
}

/// Boundedness below is preserved when replacing a function by an equal function on the domain.
theorem bounded_below_on_of_function_eq_on[Domain, Value: PartialOrder](
    domain: Set[Domain], original: Domain -> Value, replacement: Domain -> Value
) {
    function_eq_on(domain, original, replacement) and is_bounded_below_on(domain, original)
    implies is_bounded_below_on(domain, replacement)
} by {
    let bound: Value satisfy {
        is_lower_bound_on(domain, original, bound)
    }
    lower_bound_on_of_function_eq_on(domain, original, replacement, bound)
    is_lower_bound_on(domain, replacement, bound)
    exists(witness: Value) {
        is_lower_bound_on(domain, replacement, witness)
    }
    is_bounded_below_on(domain, replacement)
}

/// Boundedness is preserved when replacing a function by an equal function on the domain.
theorem bounded_on_of_function_eq_on[Domain, Value: PartialOrder](
    domain: Set[Domain], original: Domain -> Value, replacement: Domain -> Value
) {
    function_eq_on(domain, original, replacement) and is_bounded_on(domain, original)
    implies is_bounded_on(domain, replacement)
} by {
    bounded_above_on_of_function_eq_on(domain, original, replacement)
    bounded_below_on_of_function_eq_on(domain, original, replacement)
    is_bounded_above_on(domain, replacement)
    is_bounded_below_on(domain, replacement)
    is_bounded_on(domain, replacement)
}

/// Boundedness above is reflected through equality on the domain.
theorem bounded_above_on_reflect_function_eq_on[Domain, Value: PartialOrder](
    domain: Set[Domain], original: Domain -> Value, replacement: Domain -> Value
) {
    function_eq_on(domain, original, replacement) and is_bounded_above_on(domain, replacement)
    implies is_bounded_above_on(domain, original)
} by {
    function_eq_on_symm(domain, original, replacement)
    function_eq_on(domain, replacement, original)
    bounded_above_on_of_function_eq_on(domain, replacement, original)
    is_bounded_above_on(domain, original)
}

/// Boundedness below is reflected through equality on the domain.
theorem bounded_below_on_reflect_function_eq_on[Domain, Value: PartialOrder](
    domain: Set[Domain], original: Domain -> Value, replacement: Domain -> Value
) {
    function_eq_on(domain, original, replacement) and is_bounded_below_on(domain, replacement)
    implies is_bounded_below_on(domain, original)
} by {
    function_eq_on_symm(domain, original, replacement)
    function_eq_on(domain, replacement, original)
    bounded_below_on_of_function_eq_on(domain, replacement, original)
    is_bounded_below_on(domain, original)
}

/// Boundedness is reflected through equality on the domain.
theorem bounded_on_reflect_function_eq_on[Domain, Value: PartialOrder](
    domain: Set[Domain], original: Domain -> Value, replacement: Domain -> Value
) {
    function_eq_on(domain, original, replacement) and is_bounded_on(domain, replacement)
    implies is_bounded_on(domain, original)
} by {
    function_eq_on_symm(domain, original, replacement)
    function_eq_on(domain, replacement, original)
    bounded_on_of_function_eq_on(domain, replacement, original)
    is_bounded_on(domain, original)
}

/// A maximum point is preserved when replacing a function by an equal function on the domain.
theorem attains_maximum_at_of_function_eq_on[Domain, Value: PartialOrder](
    domain: Set[Domain], original: Domain -> Value, replacement: Domain -> Value, point: Domain
) {
    function_eq_on(domain, original, replacement) and attains_maximum_at(domain, original, point)
    implies attains_maximum_at(domain, replacement, point)
} by {
    domain.contains(point)
    maximum_value_is_upper_bound_on(domain, original, point)
    forall(candidate: Domain) {
        if domain.contains(candidate) {
            function_eq_on_apply(domain, original, replacement, candidate)
            function_eq_on_apply(domain, original, replacement, point)
            upper_bound_on_apply(domain, original, original(point), candidate)
            original(candidate) = replacement(candidate)
            original(point) = replacement(point)
            original(candidate) <= original(point)
            replacement(candidate) <= replacement(point)
        }
    }
    is_upper_bound_on(domain, replacement, replacement(point))
    domain.contains(point) and is_upper_bound_on(domain, replacement, replacement(point))
    attains_maximum_at(domain, replacement, point)
}

/// A minimum point is preserved when replacing a function by an equal function on the domain.
theorem attains_minimum_at_of_function_eq_on[Domain, Value: PartialOrder](
    domain: Set[Domain], original: Domain -> Value, replacement: Domain -> Value, point: Domain
) {
    function_eq_on(domain, original, replacement) and attains_minimum_at(domain, original, point)
    implies attains_minimum_at(domain, replacement, point)
} by {
    domain.contains(point)
    minimum_value_is_lower_bound_on(domain, original, point)
    forall(candidate: Domain) {
        if domain.contains(candidate) {
            function_eq_on_apply(domain, original, replacement, candidate)
            function_eq_on_apply(domain, original, replacement, point)
            lower_bound_on_apply(domain, original, original(point), candidate)
            original(candidate) = replacement(candidate)
            original(point) = replacement(point)
            original(point) <= original(candidate)
            replacement(point) <= replacement(candidate)
        }
    }
    is_lower_bound_on(domain, replacement, replacement(point))
    domain.contains(point) and is_lower_bound_on(domain, replacement, replacement(point))
    attains_minimum_at(domain, replacement, point)
}

/// A maximum point is reflected through equality on the domain.
theorem attains_maximum_at_reflect_function_eq_on[Domain, Value: PartialOrder](
    domain: Set[Domain], original: Domain -> Value, replacement: Domain -> Value, point: Domain
) {
    function_eq_on(domain, original, replacement) and attains_maximum_at(domain, replacement, point)
    implies attains_maximum_at(domain, original, point)
} by {
    function_eq_on_symm(domain, original, replacement)
    function_eq_on(domain, replacement, original)
    attains_maximum_at_of_function_eq_on(domain, replacement, original, point)
    attains_maximum_at(domain, original, point)
}

/// A minimum point is reflected through equality on the domain.
theorem attains_minimum_at_reflect_function_eq_on[Domain, Value: PartialOrder](
    domain: Set[Domain], original: Domain -> Value, replacement: Domain -> Value, point: Domain
) {
    function_eq_on(domain, original, replacement) and attains_minimum_at(domain, replacement, point)
    implies attains_minimum_at(domain, original, point)
} by {
    function_eq_on_symm(domain, original, replacement)
    function_eq_on(domain, replacement, original)
    attains_minimum_at_of_function_eq_on(domain, replacement, original, point)
    attains_minimum_at(domain, original, point)
}

/// Attainment of a maximum is preserved when replacing a function by an equal function on the domain.
theorem attains_maximum_on_of_function_eq_on[Domain, Value: PartialOrder](
    domain: Set[Domain], original: Domain -> Value, replacement: Domain -> Value
) {
    function_eq_on(domain, original, replacement) and attains_maximum_on(domain, original)
    implies attains_maximum_on(domain, replacement)
} by {
    let point: Domain satisfy {
        attains_maximum_at(domain, original, point)
    }
    attains_maximum_at_of_function_eq_on(domain, original, replacement, point)
    attains_maximum_at(domain, replacement, point)
    exists(witness: Domain) {
        attains_maximum_at(domain, replacement, witness)
    }
    attains_maximum_on(domain, replacement)
}

/// Attainment of a minimum is preserved when replacing a function by an equal function on the domain.
theorem attains_minimum_on_of_function_eq_on[Domain, Value: PartialOrder](
    domain: Set[Domain], original: Domain -> Value, replacement: Domain -> Value
) {
    function_eq_on(domain, original, replacement) and attains_minimum_on(domain, original)
    implies attains_minimum_on(domain, replacement)
} by {
    let point: Domain satisfy {
        attains_minimum_at(domain, original, point)
    }
    attains_minimum_at_of_function_eq_on(domain, original, replacement, point)
    attains_minimum_at(domain, replacement, point)
    exists(witness: Domain) {
        attains_minimum_at(domain, replacement, witness)
    }
    attains_minimum_on(domain, replacement)
}

/// Attainment of a maximum is reflected through equality on the domain.
theorem attains_maximum_on_reflect_function_eq_on[Domain, Value: PartialOrder](
    domain: Set[Domain], original: Domain -> Value, replacement: Domain -> Value
) {
    function_eq_on(domain, original, replacement) and attains_maximum_on(domain, replacement)
    implies attains_maximum_on(domain, original)
} by {
    function_eq_on_symm(domain, original, replacement)
    function_eq_on(domain, replacement, original)
    attains_maximum_on_of_function_eq_on(domain, replacement, original)
    attains_maximum_on(domain, original)
}

/// Attainment of a minimum is reflected through equality on the domain.
theorem attains_minimum_on_reflect_function_eq_on[Domain, Value: PartialOrder](
    domain: Set[Domain], original: Domain -> Value, replacement: Domain -> Value
) {
    function_eq_on(domain, original, replacement) and attains_minimum_on(domain, replacement)
    implies attains_minimum_on(domain, original)
} by {
    function_eq_on_symm(domain, original, replacement)
    function_eq_on(domain, replacement, original)
    attains_minimum_on_of_function_eq_on(domain, replacement, original)
    attains_minimum_on(domain, original)

}
