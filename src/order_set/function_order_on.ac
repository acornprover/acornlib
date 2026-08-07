/// Pointwise order comparisons for functions restricted to a subset of their domain.

from order import PartialOrder, lte_refl, lte_trans, lte_antisymm, lt_imp_lte, lt_trans,
    lt_of_lt_of_lte, lt_of_lte_of_lt
from data.basic.set import Set, sets_subset_union, union_contains_cases

/// True if two functions are equal at every point of `domain`.
define function_eq_on[Domain, Value](
    domain: Set[Domain], left: Domain -> Value, right: Domain -> Value
) -> Bool {
    forall(point: Domain) {
        domain.contains(point) implies left(point) = right(point)
    }
}

/// True if `lower` lies pointwise below `upper` on `domain`.
define function_lte_on[Domain, Value: PartialOrder](
    domain: Set[Domain], lower: Domain -> Value, upper: Domain -> Value
) -> Bool {
    forall(point: Domain) {
        domain.contains(point) implies lower(point) <= upper(point)
    }
}

/// True if `lower` lies pointwise strictly below `upper` on `domain`.
define function_lt_on[Domain, Value: PartialOrder](
    domain: Set[Domain], lower: Domain -> Value, upper: Domain -> Value
) -> Bool {
    forall(point: Domain) {
        domain.contains(point) implies lower(point) < upper(point)
    }
}

/// True if `middle` lies pointwise between `lower` and `upper` on `domain`.
define function_between_on[Domain, Value: PartialOrder](
    domain: Set[Domain], lower: Domain -> Value, middle: Domain -> Value, upper: Domain -> Value
) -> Bool {
    function_lte_on(domain, lower, middle) and function_lte_on(domain, middle, upper)
}

/// True if `middle` lies pointwise strictly between `lower` and `upper` on `domain`.
define function_strict_between_on[Domain, Value: PartialOrder](
    domain: Set[Domain], lower: Domain -> Value, middle: Domain -> Value, upper: Domain -> Value
) -> Bool {
    function_lt_on(domain, lower, middle) and function_lt_on(domain, middle, upper)
}

/// Pointwise equality on a set gives equality at any point of the set.
theorem function_eq_on_apply[Domain, Value](
    domain: Set[Domain], left: Domain -> Value, right: Domain -> Value, point: Domain
) {
    function_eq_on(domain, left, right) and domain.contains(point)
    implies left(point) = right(point)
} by {
    function_eq_on(domain, left, right) = forall(candidate: Domain) {
        domain.contains(candidate) implies left(candidate) = right(candidate)
    }
    domain.contains(point) implies left(point) = right(point)
    left(point) = right(point)
}

/// Pointwise equality on a set is symmetric.
theorem function_eq_on_symm[Domain, Value](
    domain: Set[Domain], left: Domain -> Value, right: Domain -> Value
) {
    function_eq_on(domain, left, right) implies function_eq_on(domain, right, left)
} by {
    forall(point: Domain) {
        if domain.contains(point) {
            function_eq_on_apply(domain, left, right, point)
            left(point) = right(point)
            right(point) = left(point)
        }
    }
}

/// Pointwise equality on a set is transitive.
theorem function_eq_on_trans[Domain, Value](
    domain: Set[Domain], f: Domain -> Value, g: Domain -> Value, h: Domain -> Value
) {
    function_eq_on(domain, f, g) and function_eq_on(domain, g, h) implies function_eq_on(domain, f, h)
} by {
    forall(point: Domain) {
        if domain.contains(point) {
            function_eq_on_apply(domain, f, g, point)
            function_eq_on_apply(domain, g, h, point)
            f(point) = g(point)
            g(point) = h(point)
            f(point) = h(point)
        }
    }
}

/// Pointwise equality remains true after restricting the domain.
theorem function_eq_on_subset[Domain, Value](
    subset: Set[Domain], superset: Set[Domain], left: Domain -> Value, right: Domain -> Value
) {
    subset.subset(superset) and function_eq_on(superset, left, right)
    implies function_eq_on(subset, left, right)
} by {
    forall(point: Domain) {
        if subset.contains(point) {
            subset.subset(superset) = forall(candidate: Domain) {
                subset.contains(candidate) implies superset.contains(candidate)
            }
            subset.contains(point) implies superset.contains(point)
            superset.contains(point)
            function_eq_on_apply(superset, left, right, point)
            left(point) = right(point)
        }
    }
}

/// A pointwise comparison on a set gives the comparison at any point of the set.
theorem function_lte_on_apply[Domain, Value: PartialOrder](
    domain: Set[Domain], lower: Domain -> Value, upper: Domain -> Value, point: Domain
) {
    function_lte_on(domain, lower, upper) and domain.contains(point)
    implies lower(point) <= upper(point)
} by {
    function_lte_on(domain, lower, upper) = forall(candidate: Domain) {
        domain.contains(candidate) implies lower(candidate) <= upper(candidate)
    }
    domain.contains(point) implies lower(point) <= upper(point)
    lower(point) <= upper(point)
}

/// A strict pointwise comparison on a set gives the strict comparison at any point of the set.
theorem function_lt_on_apply[Domain, Value: PartialOrder](
    domain: Set[Domain], lower: Domain -> Value, upper: Domain -> Value, point: Domain
) {
    function_lt_on(domain, lower, upper) and domain.contains(point)
    implies lower(point) < upper(point)
} by {
    function_lt_on(domain, lower, upper) = forall(candidate: Domain) {
        domain.contains(candidate) implies lower(candidate) < upper(candidate)
    }
    domain.contains(point) implies lower(point) < upper(point)
    lower(point) < upper(point)
}

/// The pointwise order on a set is reflexive.
theorem function_lte_on_refl[Domain, Value: PartialOrder](
    domain: Set[Domain], f: Domain -> Value
) {
    function_lte_on(domain, f, f)
} by {
    forall(point: Domain) {
        if domain.contains(point) {
            lte_refl(f(point))
            f(point) <= f(point)
        }
    }
}

/// Pointwise equality on a set implies pointwise order on that set.
theorem function_lte_on_of_eq_on[Domain, Value: PartialOrder](
    domain: Set[Domain], left: Domain -> Value, right: Domain -> Value
) {
    function_eq_on(domain, left, right) implies function_lte_on(domain, left, right)
} by {
    forall(point: Domain) {
        if domain.contains(point) {
            function_eq_on_apply(domain, left, right, point)
            left(point) = right(point)
            lte_refl(right(point))
            left(point) <= right(point)
        }
    }
}

/// Pointwise order on a set is transitive.
theorem function_lte_on_trans[Domain, Value: PartialOrder](
    domain: Set[Domain], f: Domain -> Value, g: Domain -> Value, h: Domain -> Value
) {
    function_lte_on(domain, f, g) and function_lte_on(domain, g, h)
    implies function_lte_on(domain, f, h)
} by {
    forall(point: Domain) {
        if domain.contains(point) {
            function_lte_on_apply(domain, f, g, point)
            function_lte_on_apply(domain, g, h, point)
            f(point) <= g(point)
            g(point) <= h(point)
            lte_trans(f(point), g(point), h(point))
            f(point) <= h(point)
        }
    }
}

/// Mutually pointwise ordered functions are pointwise equal on the set.
theorem function_eq_on_of_lte_on_both[Domain, Value: PartialOrder](
    domain: Set[Domain], left: Domain -> Value, right: Domain -> Value
) {
    function_lte_on(domain, left, right) and function_lte_on(domain, right, left)
    implies function_eq_on(domain, left, right)
} by {
    forall(point: Domain) {
        if domain.contains(point) {
            function_lte_on_apply(domain, left, right, point)
            function_lte_on_apply(domain, right, left, point)
            left(point) <= right(point)
            right(point) <= left(point)
            lte_antisymm(left(point), right(point))
            left(point) = right(point)
        }
    }
}

/// Replacing the lower function by an equal one preserves pointwise order on the set.
theorem function_lte_on_congr_left[Domain, Value: PartialOrder](
    domain: Set[Domain], f: Domain -> Value, g: Domain -> Value, h: Domain -> Value
) {
    function_eq_on(domain, f, g) and function_lte_on(domain, g, h)
    implies function_lte_on(domain, f, h)
} by {
    forall(point: Domain) {
        if domain.contains(point) {
            function_eq_on_apply(domain, f, g, point)
            function_lte_on_apply(domain, g, h, point)
            f(point) = g(point)
            g(point) <= h(point)
            f(point) <= h(point)
        }
    }
}

/// Replacing the upper function by an equal one preserves pointwise order on the set.
theorem function_lte_on_congr_right[Domain, Value: PartialOrder](
    domain: Set[Domain], f: Domain -> Value, g: Domain -> Value, h: Domain -> Value
) {
    function_lte_on(domain, f, g) and function_eq_on(domain, g, h)
    implies function_lte_on(domain, f, h)
} by {
    forall(point: Domain) {
        if domain.contains(point) {
            function_lte_on_apply(domain, f, g, point)
            function_eq_on_apply(domain, g, h, point)
            f(point) <= g(point)
            g(point) = h(point)
            f(point) <= h(point)
        }
    }
}

/// Replacing the lower function by an equal one reflects pointwise order on the set.
theorem function_lte_on_reflect_congr_left[Domain, Value: PartialOrder](
    domain: Set[Domain], f: Domain -> Value, g: Domain -> Value, h: Domain -> Value
) {
    function_eq_on(domain, f, g) and function_lte_on(domain, f, h)
    implies function_lte_on(domain, g, h)
} by {
    function_eq_on_symm(domain, f, g)
    function_eq_on(domain, g, f)
    function_lte_on_congr_left(domain, g, f, h)
    function_lte_on(domain, g, h)
}

/// Replacing the upper function by an equal one reflects pointwise order on the set.
theorem function_lte_on_reflect_congr_right[Domain, Value: PartialOrder](
    domain: Set[Domain], f: Domain -> Value, g: Domain -> Value, h: Domain -> Value
) {
    function_lte_on(domain, f, h) and function_eq_on(domain, g, h)
    implies function_lte_on(domain, f, g)
} by {
    function_eq_on_symm(domain, g, h)
    function_eq_on(domain, h, g)
    function_lte_on_congr_right(domain, f, h, g)
    function_lte_on(domain, f, g)
}

/// Replacing both sides by equal functions preserves pointwise order on the set.
theorem function_lte_on_congr[Domain, Value: PartialOrder](
    domain: Set[Domain], f: Domain -> Value, g: Domain -> Value,
    h: Domain -> Value, k: Domain -> Value
) {
    function_eq_on(domain, f, g) and function_lte_on(domain, g, h) and function_eq_on(domain, h, k)
    implies function_lte_on(domain, f, k)
} by {
    function_lte_on_congr_left(domain, f, g, h)
    function_lte_on(domain, f, h)
    function_lte_on_congr_right(domain, f, h, k)
    function_lte_on(domain, f, k)
}

/// Equal functions are pointwise ordered in both directions on the set.
theorem function_lte_on_pair_of_eq_on[Domain, Value: PartialOrder](
    domain: Set[Domain], left: Domain -> Value, right: Domain -> Value
) {
    function_eq_on(domain, left, right)
    implies function_lte_on(domain, left, right) and function_lte_on(domain, right, left)
} by {
    function_lte_on_of_eq_on(domain, left, right)
    function_eq_on_symm(domain, left, right)
    function_lte_on_of_eq_on(domain, right, left)
    function_lte_on(domain, left, right)
    function_lte_on(domain, right, left)
}

/// Pointwise strict order on a set is transitive.
theorem function_lt_on_trans[Domain, Value: PartialOrder](
    domain: Set[Domain], f: Domain -> Value, g: Domain -> Value, h: Domain -> Value
) {
    function_lt_on(domain, f, g) and function_lt_on(domain, g, h)
    implies function_lt_on(domain, f, h)
} by {
    forall(point: Domain) {
        if domain.contains(point) {
            function_lt_on_apply(domain, f, g, point)
            function_lt_on_apply(domain, g, h, point)
            f(point) < g(point)
            g(point) < h(point)
            lt_trans(f(point), g(point), h(point))
            f(point) < h(point)
        }
    }
}

/// A strict comparison followed by a non-strict comparison gives a strict comparison.
theorem function_lt_lte_on_trans[Domain, Value: PartialOrder](
    domain: Set[Domain], f: Domain -> Value, g: Domain -> Value, h: Domain -> Value
) {
    function_lt_on(domain, f, g) and function_lte_on(domain, g, h)
    implies function_lt_on(domain, f, h)
} by {
    forall(point: Domain) {
        if domain.contains(point) {
            function_lt_on_apply(domain, f, g, point)
            function_lte_on_apply(domain, g, h, point)
            f(point) < g(point)
            g(point) <= h(point)
            lt_of_lt_of_lte(f(point), g(point), h(point))
            f(point) < h(point)
        }
    }
}

/// A non-strict comparison followed by a strict comparison gives a strict comparison.
theorem function_lte_lt_on_trans[Domain, Value: PartialOrder](
    domain: Set[Domain], f: Domain -> Value, g: Domain -> Value, h: Domain -> Value
) {
    function_lte_on(domain, f, g) and function_lt_on(domain, g, h)
    implies function_lt_on(domain, f, h)
} by {
    forall(point: Domain) {
        if domain.contains(point) {
            function_lte_on_apply(domain, f, g, point)
            function_lt_on_apply(domain, g, h, point)
            f(point) <= g(point)
            g(point) < h(point)
            lt_of_lte_of_lt(f(point), g(point), h(point))
            f(point) < h(point)
        }
    }
}

/// Pointwise strict order on a set implies pointwise order on the same set.
theorem function_lt_on_imp_lte_on[Domain, Value: PartialOrder](
    domain: Set[Domain], lower: Domain -> Value, upper: Domain -> Value
) {
    function_lt_on(domain, lower, upper) implies function_lte_on(domain, lower, upper)
} by {
    forall(point: Domain) {
        if domain.contains(point) {
            function_lt_on_apply(domain, lower, upper, point)
            lower(point) < upper(point)
            lt_imp_lte(lower(point), upper(point))
            lower(point) <= upper(point)
        }
    }
}

/// Replacing the lower function by an equal one preserves strict pointwise order on the set.
theorem function_lt_on_congr_left[Domain, Value: PartialOrder](
    domain: Set[Domain], f: Domain -> Value, g: Domain -> Value, h: Domain -> Value
) {
    function_eq_on(domain, f, g) and function_lt_on(domain, g, h)
    implies function_lt_on(domain, f, h)
} by {
    forall(point: Domain) {
        if domain.contains(point) {
            function_eq_on_apply(domain, f, g, point)
            function_lt_on_apply(domain, g, h, point)
            f(point) = g(point)
            g(point) < h(point)
            f(point) < h(point)
        }
    }
}

/// Replacing the upper function by an equal one preserves strict pointwise order on the set.
theorem function_lt_on_congr_right[Domain, Value: PartialOrder](
    domain: Set[Domain], f: Domain -> Value, g: Domain -> Value, h: Domain -> Value
) {
    function_lt_on(domain, f, g) and function_eq_on(domain, g, h)
    implies function_lt_on(domain, f, h)
} by {
    forall(point: Domain) {
        if domain.contains(point) {
            function_lt_on_apply(domain, f, g, point)
            function_eq_on_apply(domain, g, h, point)
            f(point) < g(point)
            g(point) = h(point)
            f(point) < h(point)
        }
    }
}

/// Replacing the lower function by an equal one reflects strict pointwise order on the set.
theorem function_lt_on_reflect_congr_left[Domain, Value: PartialOrder](
    domain: Set[Domain], f: Domain -> Value, g: Domain -> Value, h: Domain -> Value
) {
    function_eq_on(domain, f, g) and function_lt_on(domain, f, h)
    implies function_lt_on(domain, g, h)
} by {
    function_eq_on_symm(domain, f, g)
    function_eq_on(domain, g, f)
    function_lt_on_congr_left(domain, g, f, h)
    function_lt_on(domain, g, h)
}

/// Replacing the upper function by an equal one reflects strict pointwise order on the set.
theorem function_lt_on_reflect_congr_right[Domain, Value: PartialOrder](
    domain: Set[Domain], f: Domain -> Value, g: Domain -> Value, h: Domain -> Value
) {
    function_lt_on(domain, f, h) and function_eq_on(domain, g, h)
    implies function_lt_on(domain, f, g)
} by {
    function_eq_on_symm(domain, g, h)
    function_eq_on(domain, h, g)
    function_lt_on_congr_right(domain, f, h, g)
    function_lt_on(domain, f, g)
}

/// Replacing both sides by equal functions preserves strict pointwise order on the set.
theorem function_lt_on_congr[Domain, Value: PartialOrder](
    domain: Set[Domain], f: Domain -> Value, g: Domain -> Value,
    h: Domain -> Value, k: Domain -> Value
) {
    function_eq_on(domain, f, g) and function_lt_on(domain, g, h) and function_eq_on(domain, h, k)
    implies function_lt_on(domain, f, k)
} by {
    function_lt_on_congr_left(domain, f, g, h)
    function_lt_on(domain, f, h)
    function_lt_on_congr_right(domain, f, h, k)
    function_lt_on(domain, f, k)
}

/// A pointwise between relation gives the lower comparison.
theorem function_between_on_left[Domain, Value: PartialOrder](
    domain: Set[Domain], lower: Domain -> Value, middle: Domain -> Value, upper: Domain -> Value
) {
    function_between_on(domain, lower, middle, upper) implies function_lte_on(domain, lower, middle)
} by {
    function_lte_on(domain, lower, middle)
}

/// A pointwise between relation gives the upper comparison.
theorem function_between_on_right[Domain, Value: PartialOrder](
    domain: Set[Domain], lower: Domain -> Value, middle: Domain -> Value, upper: Domain -> Value
) {
    function_between_on(domain, lower, middle, upper) implies function_lte_on(domain, middle, upper)
} by {
    function_lte_on(domain, middle, upper)
}

/// A strict pointwise between relation gives the lower strict comparison.
theorem function_strict_between_on_left[Domain, Value: PartialOrder](
    domain: Set[Domain], lower: Domain -> Value, middle: Domain -> Value, upper: Domain -> Value
) {
    function_strict_between_on(domain, lower, middle, upper) implies function_lt_on(domain, lower, middle)
} by {
    function_lt_on(domain, lower, middle)
}

/// A strict pointwise between relation gives the upper strict comparison.
theorem function_strict_between_on_right[Domain, Value: PartialOrder](
    domain: Set[Domain], lower: Domain -> Value, middle: Domain -> Value, upper: Domain -> Value
) {
    function_strict_between_on(domain, lower, middle, upper) implies function_lt_on(domain, middle, upper)
} by {
    function_lt_on(domain, middle, upper)
}

/// A strict pointwise between relation implies a non-strict pointwise between relation.
theorem function_between_on_of_strict_between_on[Domain, Value: PartialOrder](
    domain: Set[Domain], lower: Domain -> Value, middle: Domain -> Value, upper: Domain -> Value
) {
    function_strict_between_on(domain, lower, middle, upper)
    implies function_between_on(domain, lower, middle, upper)
} by {
    function_strict_between_on_left(domain, lower, middle, upper)
    function_strict_between_on_right(domain, lower, middle, upper)
    function_lt_on_imp_lte_on(domain, lower, middle)
    function_lt_on_imp_lte_on(domain, middle, upper)
    function_lte_on(domain, lower, middle)
    function_lte_on(domain, middle, upper)
    function_between_on(domain, lower, middle, upper)
}

/// A pointwise between relation remains true after restricting the domain.
theorem function_between_on_subset[Domain, Value: PartialOrder](
    subset: Set[Domain], superset: Set[Domain], lower: Domain -> Value,
    middle: Domain -> Value, upper: Domain -> Value
) {
    subset.subset(superset) and function_between_on(superset, lower, middle, upper)
    implies function_between_on(subset, lower, middle, upper)
} by {
    function_between_on_left(superset, lower, middle, upper)
    function_between_on_right(superset, lower, middle, upper)
    forall(point: Domain) {
        if subset.contains(point) {
            subset.subset(superset) = forall(candidate: Domain) {
                subset.contains(candidate) implies superset.contains(candidate)
            }
            subset.contains(point) implies superset.contains(point)
            superset.contains(point)
            function_lte_on_apply(superset, lower, middle, point)
            lower(point) <= middle(point)
        }
    }
    function_lte_on(subset, lower, middle)
    forall(point: Domain) {
        if subset.contains(point) {
            subset.subset(superset) = forall(candidate: Domain) {
                subset.contains(candidate) implies superset.contains(candidate)
            }
            subset.contains(point) implies superset.contains(point)
            superset.contains(point)
            function_lte_on_apply(superset, middle, upper, point)
            middle(point) <= upper(point)
        }
    }
    function_lte_on(subset, middle, upper)
    function_between_on(subset, lower, middle, upper)
}

/// A strict pointwise between relation remains true after restricting the domain.
theorem function_strict_between_on_subset[Domain, Value: PartialOrder](
    subset: Set[Domain], superset: Set[Domain], lower: Domain -> Value,
    middle: Domain -> Value, upper: Domain -> Value
) {
    subset.subset(superset) and function_strict_between_on(superset, lower, middle, upper)
    implies function_strict_between_on(subset, lower, middle, upper)
} by {
    function_strict_between_on_left(superset, lower, middle, upper)
    function_strict_between_on_right(superset, lower, middle, upper)
    forall(point: Domain) {
        if subset.contains(point) {
            subset.subset(superset) = forall(candidate: Domain) {
                subset.contains(candidate) implies superset.contains(candidate)
            }
            subset.contains(point) implies superset.contains(point)
            superset.contains(point)
            function_lt_on_apply(superset, lower, middle, point)
            lower(point) < middle(point)
        }
    }
    function_lt_on(subset, lower, middle)
    forall(point: Domain) {
        if subset.contains(point) {
            subset.subset(superset) = forall(candidate: Domain) {
                subset.contains(candidate) implies superset.contains(candidate)
            }
            subset.contains(point) implies superset.contains(point)
            superset.contains(point)
            function_lt_on_apply(superset, middle, upper, point)
            middle(point) < upper(point)
        }
    }
    function_lt_on(subset, middle, upper)
    function_strict_between_on(subset, lower, middle, upper)
}

/// A pointwise order comparison remains true after restricting the domain.
theorem function_lte_on_subset[Domain, Value: PartialOrder](
    subset: Set[Domain], superset: Set[Domain], lower: Domain -> Value, upper: Domain -> Value
) {
    subset.subset(superset) and function_lte_on(superset, lower, upper)
    implies function_lte_on(subset, lower, upper)
} by {
    forall(point: Domain) {
        if subset.contains(point) {
            subset.subset(superset) = forall(candidate: Domain) {
                subset.contains(candidate) implies superset.contains(candidate)
            }
            subset.contains(point) implies superset.contains(point)
            superset.contains(point)
            function_lte_on_apply(superset, lower, upper, point)
            lower(point) <= upper(point)
        }
    }
}

/// A strict pointwise comparison remains true after restricting the domain.
theorem function_lt_on_subset[Domain, Value: PartialOrder](
    subset: Set[Domain], superset: Set[Domain], lower: Domain -> Value, upper: Domain -> Value
) {
    subset.subset(superset) and function_lt_on(superset, lower, upper)
    implies function_lt_on(subset, lower, upper)
} by {
    forall(point: Domain) {
        if subset.contains(point) {
            subset.subset(superset) = forall(candidate: Domain) {
                subset.contains(candidate) implies superset.contains(candidate)
            }
            subset.contains(point) implies superset.contains(point)
            superset.contains(point)
            function_lt_on_apply(superset, lower, upper, point)
            lower(point) < upper(point)
        }
    }
}

/// Pointwise equality on two domains gives pointwise equality on their union.
theorem function_eq_on_union[Domain, Value](
    left: Set[Domain], right: Set[Domain], f: Domain -> Value, g: Domain -> Value
) {
    function_eq_on(left, f, g) and function_eq_on(right, f, g)
    implies function_eq_on(left.union(right), f, g)
} by {
    forall(point: Domain) {
        if left.union(right).contains(point) {
            union_contains_cases(left, right, point)
            if left.contains(point) {
                function_eq_on_apply(left, f, g, point)
                f(point) = g(point)
            } else {
                right.contains(point)
                function_eq_on_apply(right, f, g, point)
                f(point) = g(point)
            }
        }
    }
}

/// Pointwise equality on a union restricts to the left domain.
theorem function_eq_on_union_left[Domain, Value](
    left: Set[Domain], right: Set[Domain], f: Domain -> Value, g: Domain -> Value
) {
    function_eq_on(left.union(right), f, g) implies function_eq_on(left, f, g)
} by {
    sets_subset_union(left, right)
    left.subset(left.union(right))
    function_eq_on_subset(left, left.union(right), f, g)
    function_eq_on(left, f, g)
}

/// Pointwise equality on a union restricts to the right domain.
theorem function_eq_on_union_right[Domain, Value](
    left: Set[Domain], right: Set[Domain], f: Domain -> Value, g: Domain -> Value
) {
    function_eq_on(left.union(right), f, g) implies function_eq_on(right, f, g)
} by {
    sets_subset_union(left, right)
    right.subset(left.union(right))
    function_eq_on_subset(right, left.union(right), f, g)
    function_eq_on(right, f, g)
}

/// Pointwise order on two domains gives pointwise order on their union.
theorem function_lte_on_union[Domain, Value: PartialOrder](
    left: Set[Domain], right: Set[Domain], lower: Domain -> Value, upper: Domain -> Value
) {
    function_lte_on(left, lower, upper) and function_lte_on(right, lower, upper)
    implies function_lte_on(left.union(right), lower, upper)
} by {
    forall(point: Domain) {
        if left.union(right).contains(point) {
            union_contains_cases(left, right, point)
            if left.contains(point) {
                function_lte_on_apply(left, lower, upper, point)
                lower(point) <= upper(point)
            } else {
                right.contains(point)
                function_lte_on_apply(right, lower, upper, point)
                lower(point) <= upper(point)
            }
        }
    }
}

/// Pointwise order on a union restricts to the left domain.
theorem function_lte_on_union_left[Domain, Value: PartialOrder](
    left: Set[Domain], right: Set[Domain], lower: Domain -> Value, upper: Domain -> Value
) {
    function_lte_on(left.union(right), lower, upper) implies function_lte_on(left, lower, upper)
} by {
    sets_subset_union(left, right)
    left.subset(left.union(right))
    function_lte_on_subset(left, left.union(right), lower, upper)
    function_lte_on(left, lower, upper)
}

/// Pointwise order on a union restricts to the right domain.
theorem function_lte_on_union_right[Domain, Value: PartialOrder](
    left: Set[Domain], right: Set[Domain], lower: Domain -> Value, upper: Domain -> Value
) {
    function_lte_on(left.union(right), lower, upper) implies function_lte_on(right, lower, upper)
} by {
    sets_subset_union(left, right)
    right.subset(left.union(right))
    function_lte_on_subset(right, left.union(right), lower, upper)
    function_lte_on(right, lower, upper)
}

/// Strict pointwise order on two domains gives strict pointwise order on their union.
theorem function_lt_on_union[Domain, Value: PartialOrder](
    left: Set[Domain], right: Set[Domain], lower: Domain -> Value, upper: Domain -> Value
) {
    function_lt_on(left, lower, upper) and function_lt_on(right, lower, upper)
    implies function_lt_on(left.union(right), lower, upper)
} by {
    forall(point: Domain) {
        if left.union(right).contains(point) {
            union_contains_cases(left, right, point)
            if left.contains(point) {
                function_lt_on_apply(left, lower, upper, point)
                lower(point) < upper(point)
            } else {
                right.contains(point)
                function_lt_on_apply(right, lower, upper, point)
                lower(point) < upper(point)
            }
        }
    }
}

/// Strict pointwise order on a union restricts to the left domain.
theorem function_lt_on_union_left[Domain, Value: PartialOrder](
    left: Set[Domain], right: Set[Domain], lower: Domain -> Value, upper: Domain -> Value
) {
    function_lt_on(left.union(right), lower, upper) implies function_lt_on(left, lower, upper)
} by {
    sets_subset_union(left, right)
    left.subset(left.union(right))
    function_lt_on_subset(left, left.union(right), lower, upper)
    function_lt_on(left, lower, upper)
}

/// Strict pointwise order on a union restricts to the right domain.
theorem function_lt_on_union_right[Domain, Value: PartialOrder](
    left: Set[Domain], right: Set[Domain], lower: Domain -> Value, upper: Domain -> Value
) {
    function_lt_on(left.union(right), lower, upper) implies function_lt_on(right, lower, upper)
} by {
    sets_subset_union(left, right)
    right.subset(left.union(right))
    function_lt_on_subset(right, left.union(right), lower, upper)
    function_lt_on(right, lower, upper)
}

/// Pointwise betweenness on two domains gives pointwise betweenness on their union.
theorem function_between_on_union[Domain, Value: PartialOrder](
    left: Set[Domain], right: Set[Domain], lower: Domain -> Value,
    middle: Domain -> Value, upper: Domain -> Value
) {
    function_between_on(left, lower, middle, upper) and function_between_on(right, lower, middle, upper)
    implies function_between_on(left.union(right), lower, middle, upper)
} by {
    function_between_on_left(left, lower, middle, upper)
    function_between_on_left(right, lower, middle, upper)
    function_lte_on(left, lower, middle)
    function_lte_on(right, lower, middle)
    function_lte_on(left, lower, middle) and function_lte_on(right, lower, middle)
    function_lte_on_union(left, right, lower, middle)
    function_lte_on(left.union(right), lower, middle)
    function_between_on_right(left, lower, middle, upper)
    function_between_on_right(right, lower, middle, upper)
    function_lte_on(left, middle, upper)
    function_lte_on(right, middle, upper)
    function_lte_on(left, middle, upper) and function_lte_on(right, middle, upper)
    function_lte_on_union(left, right, middle, upper)
    function_lte_on(left.union(right), middle, upper)
    function_between_on(left.union(right), lower, middle, upper)
}

/// Pointwise betweenness on a union restricts to the left domain.
theorem function_between_on_union_left[Domain, Value: PartialOrder](
    left: Set[Domain], right: Set[Domain], lower: Domain -> Value,
    middle: Domain -> Value, upper: Domain -> Value
) {
    function_between_on(left.union(right), lower, middle, upper)
    implies function_between_on(left, lower, middle, upper)
} by {
    sets_subset_union(left, right)
    left.subset(left.union(right))
    function_between_on_subset(left, left.union(right), lower, middle, upper)
    function_between_on(left, lower, middle, upper)
}

/// Pointwise betweenness on a union restricts to the right domain.
theorem function_between_on_union_right[Domain, Value: PartialOrder](
    left: Set[Domain], right: Set[Domain], lower: Domain -> Value,
    middle: Domain -> Value, upper: Domain -> Value
) {
    function_between_on(left.union(right), lower, middle, upper)
    implies function_between_on(right, lower, middle, upper)
} by {
    sets_subset_union(left, right)
    right.subset(left.union(right))
    function_between_on_subset(right, left.union(right), lower, middle, upper)
    function_between_on(right, lower, middle, upper)
}

/// Strict pointwise betweenness on two domains gives strict pointwise betweenness on their union.
theorem function_strict_between_on_union[Domain, Value: PartialOrder](
    left: Set[Domain], right: Set[Domain], lower: Domain -> Value,
    middle: Domain -> Value, upper: Domain -> Value
) {
    function_strict_between_on(left, lower, middle, upper)
    and function_strict_between_on(right, lower, middle, upper)
    implies function_strict_between_on(left.union(right), lower, middle, upper)
} by {
    function_strict_between_on_left(left, lower, middle, upper)
    function_strict_between_on_left(right, lower, middle, upper)
    function_lt_on(left, lower, middle)
    function_lt_on(right, lower, middle)
    function_lt_on(left, lower, middle) and function_lt_on(right, lower, middle)
    function_lt_on_union(left, right, lower, middle)
    function_lt_on(left.union(right), lower, middle)
    function_strict_between_on_right(left, lower, middle, upper)
    function_strict_between_on_right(right, lower, middle, upper)
    function_lt_on(left, middle, upper)
    function_lt_on(right, middle, upper)
    function_lt_on(left, middle, upper) and function_lt_on(right, middle, upper)
    function_lt_on_union(left, right, middle, upper)
    function_lt_on(left.union(right), middle, upper)
    function_strict_between_on(left.union(right), lower, middle, upper)
}

/// Strict pointwise betweenness on a union restricts to the left domain.
theorem function_strict_between_on_union_left[Domain, Value: PartialOrder](
    left: Set[Domain], right: Set[Domain], lower: Domain -> Value,
    middle: Domain -> Value, upper: Domain -> Value
) {
    function_strict_between_on(left.union(right), lower, middle, upper)
    implies function_strict_between_on(left, lower, middle, upper)
} by {
    sets_subset_union(left, right)
    left.subset(left.union(right))
    function_strict_between_on_subset(left, left.union(right), lower, middle, upper)
    function_strict_between_on(left, lower, middle, upper)
}

/// Strict pointwise betweenness on a union restricts to the right domain.
theorem function_strict_between_on_union_right[Domain, Value: PartialOrder](
    left: Set[Domain], right: Set[Domain], lower: Domain -> Value,
    middle: Domain -> Value, upper: Domain -> Value
) {
    function_strict_between_on(left.union(right), lower, middle, upper)
    implies function_strict_between_on(right, lower, middle, upper)
} by {
    sets_subset_union(left, right)
    right.subset(left.union(right))
    function_strict_between_on_subset(right, left.union(right), lower, middle, upper)
    function_strict_between_on(right, lower, middle, upper)
}
