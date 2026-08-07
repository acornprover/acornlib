/// Intervals of a linear order as sets.

from order import LinearOrder, lte_refl, lte_trans, lt_imp_lte, lt_of_lte_of_lt, lt_of_lt_of_lte,
    lte_max_left, lte_max_right, min_lte_left, min_lte_right, max_lte_of_upper_bounds,
    max_lt_of_upper_bounds, lte_min_of_bounds, lt_min_of_bounds
from order import closed_interval, open_interval, left_open_interval, right_open_interval
from data.basic.set import Set, set_ext, intersection_contains_eq

/// The closed interval `[lower, upper]` as a set.
define closed_interval_set[L: LinearOrder](lower: L, upper: L) -> Set[L] {
    Set[L].new(closed_interval(lower, upper))
}

/// The open interval `(lower, upper)` as a set.
define open_interval_set[L: LinearOrder](lower: L, upper: L) -> Set[L] {
    Set[L].new(open_interval(lower, upper))
}

/// The interval `(lower, upper]` as a set.
define left_open_interval_set[L: LinearOrder](lower: L, upper: L) -> Set[L] {
    Set[L].new(left_open_interval(lower, upper))
}

/// The interval `[lower, upper)` as a set.
define right_open_interval_set[L: LinearOrder](lower: L, upper: L) -> Set[L] {
    Set[L].new(right_open_interval(lower, upper))
}

/// Membership in a closed interval set is the closed interval predicate.
theorem closed_interval_set_contains_eq[L: LinearOrder](lower: L, upper: L, point: L) {
    closed_interval_set(lower, upper).contains(point) = closed_interval(lower, upper, point)
}

/// Membership in an open interval set is the open interval predicate.
theorem open_interval_set_contains_eq[L: LinearOrder](lower: L, upper: L, point: L) {
    open_interval_set(lower, upper).contains(point) = open_interval(lower, upper, point)
}

/// Membership in a left-open interval set is the left-open interval predicate.
theorem left_open_interval_set_contains_eq[L: LinearOrder](lower: L, upper: L, point: L) {
    left_open_interval_set(lower, upper).contains(point) = left_open_interval(lower, upper, point)
}

/// Membership in a right-open interval set is the right-open interval predicate.
theorem right_open_interval_set_contains_eq[L: LinearOrder](lower: L, upper: L, point: L) {
    right_open_interval_set(lower, upper).contains(point) = right_open_interval(lower, upper, point)
}

/// The lower endpoint belongs to a closed interval with ordered endpoints.
theorem closed_interval_set_contains_lower[L: LinearOrder](lower: L, upper: L) {
    lower <= upper implies closed_interval_set(lower, upper).contains(lower)
} by {
    if lower <= upper {
        lte_refl(lower)
        closed_interval(lower, upper, lower)
        closed_interval_set(lower, upper).contains(lower)
    }
}

/// The upper endpoint belongs to a closed interval with ordered endpoints.
theorem closed_interval_set_contains_upper[L: LinearOrder](lower: L, upper: L) {
    lower <= upper implies closed_interval_set(lower, upper).contains(upper)
} by {
    if lower <= upper {
        lte_refl(upper)
        closed_interval(lower, upper, upper)
        closed_interval_set(lower, upper).contains(upper)
    }
}

/// The lower endpoint inequality follows from membership in a closed interval set.
theorem closed_interval_set_lower_le[L: LinearOrder](lower: L, upper: L, point: L) {
    closed_interval_set(lower, upper).contains(point) implies lower <= point
} by {
    if closed_interval_set(lower, upper).contains(point) {
        closed_interval(lower, upper, point)
        lower <= point
    }
}

/// The upper endpoint inequality follows from membership in a closed interval set.
theorem closed_interval_set_le_upper[L: LinearOrder](lower: L, upper: L, point: L) {
    closed_interval_set(lower, upper).contains(point) implies point <= upper
} by {
    if closed_interval_set(lower, upper).contains(point) {
        closed_interval(lower, upper, point)
        point <= upper
    }
}

/// The right endpoint belongs to a left-open interval when the endpoints are strictly ordered.
theorem left_open_interval_set_contains_upper[L: LinearOrder](lower: L, upper: L) {
    lower < upper implies left_open_interval_set(lower, upper).contains(upper)
} by {
    lte_refl(upper)
    left_open_interval(lower, upper, upper)
    left_open_interval_set(lower, upper).contains(upper)
}

/// The left endpoint belongs to a right-open interval when the endpoints are strictly ordered.
theorem right_open_interval_set_contains_lower[L: LinearOrder](lower: L, upper: L) {
    lower < upper implies right_open_interval_set(lower, upper).contains(lower)
} by {
    lte_refl(lower)
    right_open_interval(lower, upper, lower)
    right_open_interval_set(lower, upper).contains(lower)
}

/// The lower strict endpoint inequality follows from membership in an open interval set.
theorem open_interval_set_lower_lt[L: LinearOrder](lower: L, upper: L, point: L) {
    open_interval_set(lower, upper).contains(point) implies lower < point
} by {
    open_interval(lower, upper, point)
    lower < point
}

/// The upper strict endpoint inequality follows from membership in an open interval set.
theorem open_interval_set_lt_upper[L: LinearOrder](lower: L, upper: L, point: L) {
    open_interval_set(lower, upper).contains(point) implies point < upper
} by {
    open_interval(lower, upper, point)
    point < upper
}

/// The lower strict endpoint inequality follows from membership in a left-open interval set.
theorem left_open_interval_set_lower_lt[L: LinearOrder](lower: L, upper: L, point: L) {
    left_open_interval_set(lower, upper).contains(point) implies lower < point
} by {
    left_open_interval(lower, upper, point)
    lower < point
}

/// The upper endpoint inequality follows from membership in a left-open interval set.
theorem left_open_interval_set_le_upper[L: LinearOrder](lower: L, upper: L, point: L) {
    left_open_interval_set(lower, upper).contains(point) implies point <= upper
} by {
    left_open_interval(lower, upper, point)
    point <= upper
}

/// The lower endpoint inequality follows from membership in a right-open interval set.
theorem right_open_interval_set_lower_le[L: LinearOrder](lower: L, upper: L, point: L) {
    right_open_interval_set(lower, upper).contains(point) implies lower <= point
} by {
    right_open_interval(lower, upper, point)
    lower <= point
}

/// The upper strict endpoint inequality follows from membership in a right-open interval set.
theorem right_open_interval_set_lt_upper[L: LinearOrder](lower: L, upper: L, point: L) {
    right_open_interval_set(lower, upper).contains(point) implies point < upper
} by {
    right_open_interval(lower, upper, point)
    point < upper
}

/// Every point of an open interval belongs to the corresponding closed interval.
theorem open_interval_set_subset_closed_interval_set[L: LinearOrder](lower: L, upper: L) {
    open_interval_set(lower, upper).subset(closed_interval_set(lower, upper))
} by {
    forall(point: L) {
        if open_interval_set(lower, upper).contains(point) {
            open_interval(lower, upper, point)
            lower < point
            point < upper
            lt_imp_lte(lower, point)
            lt_imp_lte(point, upper)
            lower <= point
            point <= upper
            closed_interval(lower, upper, point)
            closed_interval_set(lower, upper).contains(point)
        }
    }
}

/// Every point of a left-open interval belongs to the corresponding closed interval.
theorem left_open_interval_set_subset_closed_interval_set[L: LinearOrder](lower: L, upper: L) {
    left_open_interval_set(lower, upper).subset(closed_interval_set(lower, upper))
} by {
    forall(point: L) {
        if left_open_interval_set(lower, upper).contains(point) {
            left_open_interval(lower, upper, point)
            lower < point
            point <= upper
            lt_imp_lte(lower, point)
            lower <= point
            closed_interval(lower, upper, point)
            closed_interval_set(lower, upper).contains(point)
        }
    }
}

/// Every point of a right-open interval belongs to the corresponding closed interval.
theorem right_open_interval_set_subset_closed_interval_set[L: LinearOrder](lower: L, upper: L) {
    right_open_interval_set(lower, upper).subset(closed_interval_set(lower, upper))
} by {
    forall(point: L) {
        if right_open_interval_set(lower, upper).contains(point) {
            right_open_interval(lower, upper, point)
            lower <= point
            point < upper
            lt_imp_lte(point, upper)
            point <= upper
            closed_interval(lower, upper, point)
            closed_interval_set(lower, upper).contains(point)
        }
    }
}

/// Every point of an open interval belongs to the corresponding left-open interval.
theorem open_interval_set_subset_left_open_interval_set[L: LinearOrder](lower: L, upper: L) {
    open_interval_set(lower, upper).subset(left_open_interval_set(lower, upper))
} by {
    forall(point: L) {
        if open_interval_set(lower, upper).contains(point) {
            open_interval(lower, upper, point)
            lower < point
            point < upper
            lt_imp_lte(point, upper)
            point <= upper
            left_open_interval(lower, upper, point)
            left_open_interval_set(lower, upper).contains(point)
        }
    }
}

/// Every point of an open interval belongs to the corresponding right-open interval.
theorem open_interval_set_subset_right_open_interval_set[L: LinearOrder](lower: L, upper: L) {
    open_interval_set(lower, upper).subset(right_open_interval_set(lower, upper))
} by {
    forall(point: L) {
        if open_interval_set(lower, upper).contains(point) {
            open_interval(lower, upper, point)
            lower < point
            point < upper
            lt_imp_lte(lower, point)
            lower <= point
            right_open_interval(lower, upper, point)
            right_open_interval_set(lower, upper).contains(point)
        }
    }
}

/// A closed interval is contained in any closed interval with weaker endpoint bounds.
theorem closed_interval_set_subset_of_bounds[L: LinearOrder](
    outer_lower: L, lower: L, upper: L, outer_upper: L
) {
    outer_lower <= lower and upper <= outer_upper
    implies closed_interval_set(lower, upper).subset(closed_interval_set(outer_lower, outer_upper))
} by {
    if outer_lower <= lower and upper <= outer_upper {
        forall(point: L) {
            if closed_interval_set(lower, upper).contains(point) {
                closed_interval(lower, upper, point)
                lower <= point
                point <= upper
                lte_trans(outer_lower, lower, point)
                outer_lower <= point
                lte_trans(point, upper, outer_upper)
                point <= outer_upper
                closed_interval(outer_lower, outer_upper, point)
                closed_interval_set(outer_lower, outer_upper).contains(point)
            }
        }
    }
}

/// An open interval is contained in any open interval with weaker endpoint bounds.
theorem open_interval_set_subset_of_bounds[L: LinearOrder](
    outer_lower: L, lower: L, upper: L, outer_upper: L
) {
    outer_lower <= lower and upper <= outer_upper
    implies open_interval_set(lower, upper).subset(open_interval_set(outer_lower, outer_upper))
} by {
    forall(point: L) {
        if open_interval_set(lower, upper).contains(point) {
            open_interval(lower, upper, point)
            lower < point
            point < upper
            lt_of_lte_of_lt(outer_lower, lower, point)
            outer_lower < point
            lt_of_lt_of_lte(point, upper, outer_upper)
            point < outer_upper
            open_interval(outer_lower, outer_upper, point)
            open_interval_set(outer_lower, outer_upper).contains(point)
        }
    }
}

/// A left-open interval is contained in any left-open interval with weaker endpoint bounds.
theorem left_open_interval_set_subset_of_bounds[L: LinearOrder](
    outer_lower: L, lower: L, upper: L, outer_upper: L
) {
    outer_lower <= lower and upper <= outer_upper
    implies left_open_interval_set(lower, upper).subset(left_open_interval_set(outer_lower, outer_upper))
} by {
    forall(point: L) {
        if left_open_interval_set(lower, upper).contains(point) {
            left_open_interval(lower, upper, point)
            lower < point
            point <= upper
            lt_of_lte_of_lt(outer_lower, lower, point)
            outer_lower < point
            lte_trans(point, upper, outer_upper)
            point <= outer_upper
            left_open_interval(outer_lower, outer_upper, point)
            left_open_interval_set(outer_lower, outer_upper).contains(point)
        }
    }
}

/// A right-open interval is contained in any right-open interval with weaker endpoint bounds.
theorem right_open_interval_set_subset_of_bounds[L: LinearOrder](
    outer_lower: L, lower: L, upper: L, outer_upper: L
) {
    outer_lower <= lower and upper <= outer_upper
    implies right_open_interval_set(lower, upper).subset(right_open_interval_set(outer_lower, outer_upper))
} by {
    forall(point: L) {
        if right_open_interval_set(lower, upper).contains(point) {
            right_open_interval(lower, upper, point)
            lower <= point
            point < upper
            lte_trans(outer_lower, lower, point)
            outer_lower <= point
            lt_of_lt_of_lte(point, upper, outer_upper)
            point < outer_upper
            right_open_interval(outer_lower, outer_upper, point)
            right_open_interval_set(outer_lower, outer_upper).contains(point)
        }
    }
}

/// An open interval is contained in any closed interval with weaker endpoint bounds.
theorem open_interval_set_subset_closed_interval_set_of_bounds[L: LinearOrder](
    outer_lower: L, lower: L, upper: L, outer_upper: L
) {
    outer_lower <= lower and upper <= outer_upper
    implies open_interval_set(lower, upper).subset(closed_interval_set(outer_lower, outer_upper))
} by {
    forall(point: L) {
        if open_interval_set(lower, upper).contains(point) {
            open_interval(lower, upper, point)
            lower < point
            point < upper
            lt_imp_lte(lower, point)
            lower <= point
            lte_trans(outer_lower, lower, point)
            outer_lower <= point
            lt_imp_lte(point, upper)
            point <= upper
            lte_trans(point, upper, outer_upper)
            point <= outer_upper
            closed_interval(outer_lower, outer_upper, point)
            closed_interval_set(outer_lower, outer_upper).contains(point)
        }
    }
}

/// An open interval is contained in any left-open interval with weaker endpoint bounds.
theorem open_interval_set_subset_left_open_interval_set_of_bounds[L: LinearOrder](
    outer_lower: L, lower: L, upper: L, outer_upper: L
) {
    outer_lower <= lower and upper <= outer_upper
    implies open_interval_set(lower, upper).subset(left_open_interval_set(outer_lower, outer_upper))
} by {
    forall(point: L) {
        if open_interval_set(lower, upper).contains(point) {
            open_interval(lower, upper, point)
            lower < point
            point < upper
            lt_of_lte_of_lt(outer_lower, lower, point)
            outer_lower < point
            lt_imp_lte(point, upper)
            point <= upper
            lte_trans(point, upper, outer_upper)
            point <= outer_upper
            left_open_interval(outer_lower, outer_upper, point)
            left_open_interval_set(outer_lower, outer_upper).contains(point)
        }
    }
}

/// An open interval is contained in any right-open interval with weaker endpoint bounds.
theorem open_interval_set_subset_right_open_interval_set_of_bounds[L: LinearOrder](
    outer_lower: L, lower: L, upper: L, outer_upper: L
) {
    outer_lower <= lower and upper <= outer_upper
    implies open_interval_set(lower, upper).subset(right_open_interval_set(outer_lower, outer_upper))
} by {
    forall(point: L) {
        if open_interval_set(lower, upper).contains(point) {
            open_interval(lower, upper, point)
            lower < point
            point < upper
            lt_imp_lte(lower, point)
            lower <= point
            lte_trans(outer_lower, lower, point)
            outer_lower <= point
            lt_of_lt_of_lte(point, upper, outer_upper)
            point < outer_upper
            right_open_interval(outer_lower, outer_upper, point)
            right_open_interval_set(outer_lower, outer_upper).contains(point)
        }
    }
}

/// A left-open interval is contained in any closed interval with weaker endpoint bounds.
theorem left_open_interval_set_subset_closed_interval_set_of_bounds[L: LinearOrder](
    outer_lower: L, lower: L, upper: L, outer_upper: L
) {
    outer_lower <= lower and upper <= outer_upper
    implies left_open_interval_set(lower, upper).subset(closed_interval_set(outer_lower, outer_upper))
} by {
    forall(point: L) {
        if left_open_interval_set(lower, upper).contains(point) {
            left_open_interval(lower, upper, point)
            lower < point
            point <= upper
            lt_imp_lte(lower, point)
            lower <= point
            lte_trans(outer_lower, lower, point)
            outer_lower <= point
            lte_trans(point, upper, outer_upper)
            point <= outer_upper
            closed_interval(outer_lower, outer_upper, point)
            closed_interval_set(outer_lower, outer_upper).contains(point)
        }
    }
}

/// A right-open interval is contained in any closed interval with weaker endpoint bounds.
theorem right_open_interval_set_subset_closed_interval_set_of_bounds[L: LinearOrder](
    outer_lower: L, lower: L, upper: L, outer_upper: L
) {
    outer_lower <= lower and upper <= outer_upper
    implies right_open_interval_set(lower, upper).subset(closed_interval_set(outer_lower, outer_upper))
} by {
    forall(point: L) {
        if right_open_interval_set(lower, upper).contains(point) {
            right_open_interval(lower, upper, point)
            lower <= point
            point < upper
            lte_trans(outer_lower, lower, point)
            outer_lower <= point
            lt_imp_lte(point, upper)
            point <= upper
            lte_trans(point, upper, outer_upper)
            point <= outer_upper
            closed_interval(outer_lower, outer_upper, point)
            closed_interval_set(outer_lower, outer_upper).contains(point)
        }
    }
}

/// A left-open interval is contained in a right-open interval with a strictly weaker upper bound.
theorem left_open_interval_set_subset_right_open_interval_set_of_upper_strict_bound[L: LinearOrder](
    outer_lower: L, lower: L, upper: L, outer_upper: L
) {
    outer_lower <= lower and upper < outer_upper
    implies left_open_interval_set(lower, upper).subset(right_open_interval_set(outer_lower, outer_upper))
} by {
    forall(point: L) {
        if left_open_interval_set(lower, upper).contains(point) {
            left_open_interval(lower, upper, point)
            lower < point
            point <= upper
            lt_imp_lte(lower, point)
            lower <= point
            lte_trans(outer_lower, lower, point)
            outer_lower <= point
            lt_of_lte_of_lt(point, upper, outer_upper)
            point < outer_upper
            right_open_interval(outer_lower, outer_upper, point)
            right_open_interval_set(outer_lower, outer_upper).contains(point)
        }
    }
}

/// A right-open interval is contained in a left-open interval with a strictly weaker lower bound.
theorem right_open_interval_set_subset_left_open_interval_set_of_lower_strict_bound[L: LinearOrder](
    outer_lower: L, lower: L, upper: L, outer_upper: L
) {
    outer_lower < lower and upper <= outer_upper
    implies right_open_interval_set(lower, upper).subset(left_open_interval_set(outer_lower, outer_upper))
} by {
    forall(point: L) {
        if right_open_interval_set(lower, upper).contains(point) {
            right_open_interval(lower, upper, point)
            lower <= point
            point < upper
            lt_of_lt_of_lte(outer_lower, lower, point)
            outer_lower < point
            lt_imp_lte(point, upper)
            point <= upper
            lte_trans(point, upper, outer_upper)
            point <= outer_upper
            left_open_interval(outer_lower, outer_upper, point)
            left_open_interval_set(outer_lower, outer_upper).contains(point)
        }
    }
}

/// A closed interval is contained in an open interval whose bounds are strictly weaker.
theorem closed_interval_set_subset_open_interval_set_of_strict_bounds[L: LinearOrder](
    outer_lower: L, lower: L, upper: L, outer_upper: L
) {
    outer_lower < lower and upper < outer_upper
    implies closed_interval_set(lower, upper).subset(open_interval_set(outer_lower, outer_upper))
} by {
    forall(point: L) {
        if closed_interval_set(lower, upper).contains(point) {
            closed_interval(lower, upper, point)
            lower <= point
            point <= upper
            lt_of_lt_of_lte(outer_lower, lower, point)
            outer_lower < point
            lt_of_lte_of_lt(point, upper, outer_upper)
            point < outer_upper
            open_interval(outer_lower, outer_upper, point)
            open_interval_set(outer_lower, outer_upper).contains(point)
        }
    }
}

/// A closed interval is contained in a left-open interval with a strictly weaker lower bound.
theorem closed_interval_set_subset_left_open_interval_set_of_lower_strict_bound[L: LinearOrder](
    outer_lower: L, lower: L, upper: L, outer_upper: L
) {
    outer_lower < lower and upper <= outer_upper
    implies closed_interval_set(lower, upper).subset(left_open_interval_set(outer_lower, outer_upper))
} by {
    forall(point: L) {
        if closed_interval_set(lower, upper).contains(point) {
            closed_interval(lower, upper, point)
            lower <= point
            point <= upper
            lt_of_lt_of_lte(outer_lower, lower, point)
            outer_lower < point
            lte_trans(point, upper, outer_upper)
            point <= outer_upper
            left_open_interval(outer_lower, outer_upper, point)
            left_open_interval_set(outer_lower, outer_upper).contains(point)
        }
    }
}

/// A closed interval is contained in a right-open interval with a strictly weaker upper bound.
theorem closed_interval_set_subset_right_open_interval_set_of_upper_strict_bound[L: LinearOrder](
    outer_lower: L, lower: L, upper: L, outer_upper: L
) {
    outer_lower <= lower and upper < outer_upper
    implies closed_interval_set(lower, upper).subset(right_open_interval_set(outer_lower, outer_upper))
} by {
    forall(point: L) {
        if closed_interval_set(lower, upper).contains(point) {
            closed_interval(lower, upper, point)
            lower <= point
            point <= upper
            lte_trans(outer_lower, lower, point)
            outer_lower <= point
            lt_of_lte_of_lt(point, upper, outer_upper)
            point < outer_upper
            right_open_interval(outer_lower, outer_upper, point)
            right_open_interval_set(outer_lower, outer_upper).contains(point)
        }
    }
}

/// A left-open interval is contained in an open interval with a strictly weaker upper bound.
theorem left_open_interval_set_subset_open_interval_set_of_upper_strict_bound[L: LinearOrder](
    outer_lower: L, lower: L, upper: L, outer_upper: L
) {
    outer_lower <= lower and upper < outer_upper
    implies left_open_interval_set(lower, upper).subset(open_interval_set(outer_lower, outer_upper))
} by {
    forall(point: L) {
        if left_open_interval_set(lower, upper).contains(point) {
            left_open_interval(lower, upper, point)
            lower < point
            point <= upper
            lt_of_lte_of_lt(outer_lower, lower, point)
            outer_lower < point
            lt_of_lte_of_lt(point, upper, outer_upper)
            point < outer_upper
            open_interval(outer_lower, outer_upper, point)
            open_interval_set(outer_lower, outer_upper).contains(point)
        }
    }
}

/// A right-open interval is contained in an open interval with a strictly weaker lower bound.
theorem right_open_interval_set_subset_open_interval_set_of_lower_strict_bound[L: LinearOrder](
    outer_lower: L, lower: L, upper: L, outer_upper: L
) {
    outer_lower < lower and upper <= outer_upper
    implies right_open_interval_set(lower, upper).subset(open_interval_set(outer_lower, outer_upper))
} by {
    forall(point: L) {
        if right_open_interval_set(lower, upper).contains(point) {
            right_open_interval(lower, upper, point)
            lower <= point
            point < upper
            lt_of_lt_of_lte(outer_lower, lower, point)
            outer_lower < point
            lt_of_lt_of_lte(point, upper, outer_upper)
            point < outer_upper
            open_interval(outer_lower, outer_upper, point)
            open_interval_set(outer_lower, outer_upper).contains(point)
        }
    }
}

/// The intersection of two closed interval sets is the closed interval set with joined lower endpoints and met upper endpoints.
theorem closed_interval_set_intersection_eq[L: LinearOrder](a1: L, b1: L, a2: L, b2: L) {
    closed_interval_set(a1, b1).intersection(closed_interval_set(a2, b2)) =
    closed_interval_set(a1.max(a2), b1.min(b2))
} by {
    forall(point: L) {
        intersection_contains_eq(closed_interval_set(a1, b1), closed_interval_set(a2, b2), point)
        closed_interval_set_contains_eq(a1, b1, point)
        closed_interval_set_contains_eq(a2, b2, point)
        closed_interval_set_contains_eq(a1.max(a2), b1.min(b2), point)
        if closed_interval_set(a1, b1).intersection(closed_interval_set(a2, b2)).contains(point) {
            closed_interval(a1, b1, point)
            closed_interval(a2, b2, point)
            a1 <= point
            a2 <= point
            max_lte_of_upper_bounds(a1, a2, point)
            a1.max(a2) <= point
            point <= b1
            point <= b2
            lte_min_of_bounds(point, b1, b2)
            point <= b1.min(b2)
            closed_interval(a1.max(a2), b1.min(b2), point)
            closed_interval_set(a1.max(a2), b1.min(b2)).contains(point)
        }
        if closed_interval_set(a1.max(a2), b1.min(b2)).contains(point) {
            closed_interval(a1.max(a2), b1.min(b2), point)
            a1.max(a2) <= point
            lte_max_left(a1, a2)
            a1 <= a1.max(a2)
            lte_trans(a1, a1.max(a2), point)
            a1 <= point
            lte_max_right(a1, a2)
            a2 <= a1.max(a2)
            lte_trans(a2, a1.max(a2), point)
            a2 <= point
            point <= b1.min(b2)
            min_lte_left(b1, b2)
            b1.min(b2) <= b1
            lte_trans(point, b1.min(b2), b1)
            point <= b1
            min_lte_right(b1, b2)
            b1.min(b2) <= b2
            lte_trans(point, b1.min(b2), b2)
            point <= b2
            closed_interval(a1, b1, point)
            closed_interval(a2, b2, point)
            closed_interval_set(a1, b1).contains(point)
            closed_interval_set(a2, b2).contains(point)
            closed_interval_set(a1, b1).intersection(closed_interval_set(a2, b2)).contains(point)
        }
        closed_interval_set(a1, b1).intersection(closed_interval_set(a2, b2)).contains(point) =
        closed_interval_set(a1.max(a2), b1.min(b2)).contains(point)
    }
    set_ext(closed_interval_set(a1, b1).intersection(closed_interval_set(a2, b2)),
        closed_interval_set(a1.max(a2), b1.min(b2)))
}

/// The intersection of two open interval sets is the open interval set with joined lower endpoints and met upper endpoints.
theorem open_interval_set_intersection_eq[L: LinearOrder](a1: L, b1: L, a2: L, b2: L) {
    open_interval_set(a1, b1).intersection(open_interval_set(a2, b2)) =
    open_interval_set(a1.max(a2), b1.min(b2))
} by {
    forall(point: L) {
        intersection_contains_eq(open_interval_set(a1, b1), open_interval_set(a2, b2), point)
        open_interval_set_contains_eq(a1, b1, point)
        open_interval_set_contains_eq(a2, b2, point)
        open_interval_set_contains_eq(a1.max(a2), b1.min(b2), point)
        if open_interval_set(a1, b1).intersection(open_interval_set(a2, b2)).contains(point) {
            open_interval(a1, b1, point)
            open_interval(a2, b2, point)
            a1 < point
            a2 < point
            max_lt_of_upper_bounds(a1, a2, point)
            a1.max(a2) < point
            point < b1
            point < b2
            lt_min_of_bounds(point, b1, b2)
            point < b1.min(b2)
            open_interval(a1.max(a2), b1.min(b2), point)
            open_interval_set(a1.max(a2), b1.min(b2)).contains(point)
        }
        if open_interval_set(a1.max(a2), b1.min(b2)).contains(point) {
            open_interval(a1.max(a2), b1.min(b2), point)
            a1.max(a2) < point
            lte_max_left(a1, a2)
            a1 <= a1.max(a2)
            lt_of_lte_of_lt(a1, a1.max(a2), point)
            a1 < point
            lte_max_right(a1, a2)
            a2 <= a1.max(a2)
            lt_of_lte_of_lt(a2, a1.max(a2), point)
            a2 < point
            point < b1.min(b2)
            min_lte_left(b1, b2)
            b1.min(b2) <= b1
            lt_of_lt_of_lte(point, b1.min(b2), b1)
            point < b1
            min_lte_right(b1, b2)
            b1.min(b2) <= b2
            lt_of_lt_of_lte(point, b1.min(b2), b2)
            point < b2
            open_interval(a1, b1, point)
            open_interval(a2, b2, point)
            open_interval_set(a1, b1).contains(point)
            open_interval_set(a2, b2).contains(point)
            open_interval_set(a1, b1).intersection(open_interval_set(a2, b2)).contains(point)
        }
        open_interval_set(a1, b1).intersection(open_interval_set(a2, b2)).contains(point) =
        open_interval_set(a1.max(a2), b1.min(b2)).contains(point)
    }
    set_ext(open_interval_set(a1, b1).intersection(open_interval_set(a2, b2)),
        open_interval_set(a1.max(a2), b1.min(b2)))
}

/// The intersection of two left-open interval sets is the left-open interval set with joined lower endpoints and met upper endpoints.
theorem left_open_interval_set_intersection_eq[L: LinearOrder](a1: L, b1: L, a2: L, b2: L) {
    left_open_interval_set(a1, b1).intersection(left_open_interval_set(a2, b2)) =
    left_open_interval_set(a1.max(a2), b1.min(b2))
} by {
    forall(point: L) {
        intersection_contains_eq(left_open_interval_set(a1, b1), left_open_interval_set(a2, b2), point)
        left_open_interval_set_contains_eq(a1, b1, point)
        left_open_interval_set_contains_eq(a2, b2, point)
        left_open_interval_set_contains_eq(a1.max(a2), b1.min(b2), point)
        if left_open_interval_set(a1, b1).intersection(left_open_interval_set(a2, b2)).contains(point) {
            left_open_interval(a1, b1, point)
            left_open_interval(a2, b2, point)
            a1 < point
            a2 < point
            max_lt_of_upper_bounds(a1, a2, point)
            a1.max(a2) < point
            point <= b1
            point <= b2
            lte_min_of_bounds(point, b1, b2)
            point <= b1.min(b2)
            left_open_interval(a1.max(a2), b1.min(b2), point)
            left_open_interval_set(a1.max(a2), b1.min(b2)).contains(point)
        }
        if left_open_interval_set(a1.max(a2), b1.min(b2)).contains(point) {
            left_open_interval(a1.max(a2), b1.min(b2), point)
            a1.max(a2) < point
            lte_max_left(a1, a2)
            a1 <= a1.max(a2)
            lt_of_lte_of_lt(a1, a1.max(a2), point)
            a1 < point
            lte_max_right(a1, a2)
            a2 <= a1.max(a2)
            lt_of_lte_of_lt(a2, a1.max(a2), point)
            a2 < point
            point <= b1.min(b2)
            min_lte_left(b1, b2)
            b1.min(b2) <= b1
            lte_trans(point, b1.min(b2), b1)
            point <= b1
            min_lte_right(b1, b2)
            b1.min(b2) <= b2
            lte_trans(point, b1.min(b2), b2)
            point <= b2
            left_open_interval(a1, b1, point)
            left_open_interval(a2, b2, point)
            left_open_interval_set(a1, b1).contains(point)
            left_open_interval_set(a2, b2).contains(point)
            left_open_interval_set(a1, b1).intersection(left_open_interval_set(a2, b2)).contains(point)
        }
        left_open_interval_set(a1, b1).intersection(left_open_interval_set(a2, b2)).contains(point) =
        left_open_interval_set(a1.max(a2), b1.min(b2)).contains(point)
    }
    set_ext(left_open_interval_set(a1, b1).intersection(left_open_interval_set(a2, b2)),
        left_open_interval_set(a1.max(a2), b1.min(b2)))
}

/// The intersection of two right-open interval sets is the right-open interval set with joined lower endpoints and met upper endpoints.
theorem right_open_interval_set_intersection_eq[L: LinearOrder](a1: L, b1: L, a2: L, b2: L) {
    right_open_interval_set(a1, b1).intersection(right_open_interval_set(a2, b2)) =
    right_open_interval_set(a1.max(a2), b1.min(b2))
} by {
    forall(point: L) {
        intersection_contains_eq(right_open_interval_set(a1, b1), right_open_interval_set(a2, b2), point)
        right_open_interval_set_contains_eq(a1, b1, point)
        right_open_interval_set_contains_eq(a2, b2, point)
        right_open_interval_set_contains_eq(a1.max(a2), b1.min(b2), point)
        if right_open_interval_set(a1, b1).intersection(right_open_interval_set(a2, b2)).contains(point) {
            right_open_interval(a1, b1, point)
            right_open_interval(a2, b2, point)
            a1 <= point
            a2 <= point
            max_lte_of_upper_bounds(a1, a2, point)
            a1.max(a2) <= point
            point < b1
            point < b2
            lt_min_of_bounds(point, b1, b2)
            point < b1.min(b2)
            right_open_interval(a1.max(a2), b1.min(b2), point)
            right_open_interval_set(a1.max(a2), b1.min(b2)).contains(point)
        }
        if right_open_interval_set(a1.max(a2), b1.min(b2)).contains(point) {
            right_open_interval(a1.max(a2), b1.min(b2), point)
            a1.max(a2) <= point
            lte_max_left(a1, a2)
            a1 <= a1.max(a2)
            lte_trans(a1, a1.max(a2), point)
            a1 <= point
            lte_max_right(a1, a2)
            a2 <= a1.max(a2)
            lte_trans(a2, a1.max(a2), point)
            a2 <= point
            point < b1.min(b2)
            min_lte_left(b1, b2)
            b1.min(b2) <= b1
            lt_of_lt_of_lte(point, b1.min(b2), b1)
            point < b1
            min_lte_right(b1, b2)
            b1.min(b2) <= b2
            lt_of_lt_of_lte(point, b1.min(b2), b2)
            point < b2
            right_open_interval(a1, b1, point)
            right_open_interval(a2, b2, point)
            right_open_interval_set(a1, b1).contains(point)
            right_open_interval_set(a2, b2).contains(point)
            right_open_interval_set(a1, b1).intersection(right_open_interval_set(a2, b2)).contains(point)
        }
        right_open_interval_set(a1, b1).intersection(right_open_interval_set(a2, b2)).contains(point) =
        right_open_interval_set(a1.max(a2), b1.min(b2)).contains(point)
    }
    set_ext(right_open_interval_set(a1, b1).intersection(right_open_interval_set(a2, b2)),
        right_open_interval_set(a1.max(a2), b1.min(b2)))
}
