/// Bounds for monotone maps on closed intervals.

from order import LinearOrder, PartialOrder
from order_set.interval_set import closed_interval_set, open_interval_set, left_open_interval_set,
    right_open_interval_set,
    closed_interval_set_contains_lower, closed_interval_set_contains_upper,
    left_open_interval_set_contains_upper, right_open_interval_set_contains_lower,
    closed_interval_set_lower_le, closed_interval_set_le_upper,
    left_open_interval_set_le_upper, right_open_interval_set_lower_le,
    open_interval_set_subset_closed_interval_set, left_open_interval_set_subset_closed_interval_set,
    right_open_interval_set_subset_closed_interval_set
from order_set.maps_on import is_monotone_on, is_antitone_on, monotone_on_apply, antitone_on_apply
from order_set.function_bounds import is_upper_bound_on, is_lower_bound_on, is_bounded_above_on,
    is_bounded_below_on, is_bounded_on, attains_maximum_at, attains_minimum_at,
    attains_maximum_on, attains_minimum_on, upper_bound_on_subset, lower_bound_on_subset,
    attains_maximum_imp_bounded_above, attains_minimum_imp_bounded_below

/// A monotone map on `[lower, upper]` is bounded above by its value at `upper`.
theorem monotone_on_closed_interval_upper_bound[Domain: LinearOrder, Value: PartialOrder](
    lower: Domain, upper: Domain, f: Domain -> Value
) {
    lower <= upper and is_monotone_on(closed_interval_set(lower, upper), f)
    implies is_upper_bound_on(closed_interval_set(lower, upper), f, f(upper))
} by {
    if lower <= upper and is_monotone_on(closed_interval_set(lower, upper), f) {
        closed_interval_set_contains_upper(lower, upper)
        closed_interval_set(lower, upper).contains(upper)
        forall(point: Domain) {
            if closed_interval_set(lower, upper).contains(point) {
                closed_interval_set_le_upper(lower, upper, point)
                point <= upper
                monotone_on_apply(closed_interval_set(lower, upper), f, point, upper)
                f(point) <= f(upper)
            }
        }
        is_upper_bound_on(closed_interval_set(lower, upper), f, f(upper))
    }
}

/// A monotone map on `[lower, upper]` is bounded below by its value at `lower`.
theorem monotone_on_closed_interval_lower_bound[Domain: LinearOrder, Value: PartialOrder](
    lower: Domain, upper: Domain, f: Domain -> Value
) {
    lower <= upper and is_monotone_on(closed_interval_set(lower, upper), f)
    implies is_lower_bound_on(closed_interval_set(lower, upper), f, f(lower))
} by {
    if lower <= upper and is_monotone_on(closed_interval_set(lower, upper), f) {
        closed_interval_set_contains_lower(lower, upper)
        closed_interval_set(lower, upper).contains(lower)
        forall(point: Domain) {
            if closed_interval_set(lower, upper).contains(point) {
                closed_interval_set_lower_le(lower, upper, point)
                lower <= point
                monotone_on_apply(closed_interval_set(lower, upper), f, lower, point)
                f(lower) <= f(point)
            }
        }
        is_lower_bound_on(closed_interval_set(lower, upper), f, f(lower))
    }
}

/// A monotone map on `[lower, upper]` is bounded above.
theorem monotone_on_closed_interval_bounded_above[Domain: LinearOrder, Value: PartialOrder](
    lower: Domain, upper: Domain, f: Domain -> Value
) {
    lower <= upper and is_monotone_on(closed_interval_set(lower, upper), f)
    implies is_bounded_above_on(closed_interval_set(lower, upper), f)
} by {
    if lower <= upper and is_monotone_on(closed_interval_set(lower, upper), f) {
        monotone_on_closed_interval_upper_bound(lower, upper, f)
        is_upper_bound_on(closed_interval_set(lower, upper), f, f(upper))
        exists(bound: Value) {
            is_upper_bound_on(closed_interval_set(lower, upper), f, bound)
        }
        is_bounded_above_on(closed_interval_set(lower, upper), f)
    }
}

/// A monotone map on `[lower, upper]` is bounded below.
theorem monotone_on_closed_interval_bounded_below[Domain: LinearOrder, Value: PartialOrder](
    lower: Domain, upper: Domain, f: Domain -> Value
) {
    lower <= upper and is_monotone_on(closed_interval_set(lower, upper), f)
    implies is_bounded_below_on(closed_interval_set(lower, upper), f)
} by {
    if lower <= upper and is_monotone_on(closed_interval_set(lower, upper), f) {
        monotone_on_closed_interval_lower_bound(lower, upper, f)
        is_lower_bound_on(closed_interval_set(lower, upper), f, f(lower))
        exists(bound: Value) {
            is_lower_bound_on(closed_interval_set(lower, upper), f, bound)
        }
        is_bounded_below_on(closed_interval_set(lower, upper), f)
    }
}

/// A monotone map on `[lower, upper]` is bounded.
theorem monotone_on_closed_interval_bounded[Domain: LinearOrder, Value: PartialOrder](
    lower: Domain, upper: Domain, f: Domain -> Value
) {
    lower <= upper and is_monotone_on(closed_interval_set(lower, upper), f)
    implies is_bounded_on(closed_interval_set(lower, upper), f)
} by {
    if lower <= upper and is_monotone_on(closed_interval_set(lower, upper), f) {
        monotone_on_closed_interval_bounded_above(lower, upper, f)
        monotone_on_closed_interval_bounded_below(lower, upper, f)
        is_bounded_above_on(closed_interval_set(lower, upper), f)
        is_bounded_below_on(closed_interval_set(lower, upper), f)
        is_bounded_on(closed_interval_set(lower, upper), f)
    }
}

/// A monotone map on `[lower, upper]` is bounded above on `(lower, upper)` by its value at `upper`.
theorem monotone_on_open_interval_upper_bound_from_closed_interval[Domain: LinearOrder, Value: PartialOrder](
    lower: Domain, upper: Domain, f: Domain -> Value
) {
    lower <= upper and is_monotone_on(closed_interval_set(lower, upper), f)
    implies is_upper_bound_on(open_interval_set(lower, upper), f, f(upper))
} by {
    monotone_on_closed_interval_upper_bound(lower, upper, f)
    is_upper_bound_on(closed_interval_set(lower, upper), f, f(upper))
    open_interval_set_subset_closed_interval_set(lower, upper)
    upper_bound_on_subset(open_interval_set(lower, upper), closed_interval_set(lower, upper), f, f(upper))
    is_upper_bound_on(open_interval_set(lower, upper), f, f(upper))
}

/// A monotone map on `[lower, upper]` is bounded below on `(lower, upper)` by its value at `lower`.
theorem monotone_on_open_interval_lower_bound_from_closed_interval[Domain: LinearOrder, Value: PartialOrder](
    lower: Domain, upper: Domain, f: Domain -> Value
) {
    lower <= upper and is_monotone_on(closed_interval_set(lower, upper), f)
    implies is_lower_bound_on(open_interval_set(lower, upper), f, f(lower))
} by {
    monotone_on_closed_interval_lower_bound(lower, upper, f)
    is_lower_bound_on(closed_interval_set(lower, upper), f, f(lower))
    open_interval_set_subset_closed_interval_set(lower, upper)
    lower_bound_on_subset(open_interval_set(lower, upper), closed_interval_set(lower, upper), f, f(lower))
    is_lower_bound_on(open_interval_set(lower, upper), f, f(lower))
}

/// A monotone map on `[lower, upper]` is bounded above on `(lower, upper)`.
theorem monotone_on_open_interval_bounded_above_from_closed_interval[Domain: LinearOrder, Value: PartialOrder](
    lower: Domain, upper: Domain, f: Domain -> Value
) {
    lower <= upper and is_monotone_on(closed_interval_set(lower, upper), f)
    implies is_bounded_above_on(open_interval_set(lower, upper), f)
} by {
    monotone_on_open_interval_upper_bound_from_closed_interval(lower, upper, f)
    is_upper_bound_on(open_interval_set(lower, upper), f, f(upper))
    exists(bound: Value) {
        is_upper_bound_on(open_interval_set(lower, upper), f, bound)
    }
    is_bounded_above_on(open_interval_set(lower, upper), f)
}

/// A monotone map on `[lower, upper]` is bounded below on `(lower, upper)`.
theorem monotone_on_open_interval_bounded_below_from_closed_interval[Domain: LinearOrder, Value: PartialOrder](
    lower: Domain, upper: Domain, f: Domain -> Value
) {
    lower <= upper and is_monotone_on(closed_interval_set(lower, upper), f)
    implies is_bounded_below_on(open_interval_set(lower, upper), f)
} by {
    monotone_on_open_interval_lower_bound_from_closed_interval(lower, upper, f)
    is_lower_bound_on(open_interval_set(lower, upper), f, f(lower))
    exists(bound: Value) {
        is_lower_bound_on(open_interval_set(lower, upper), f, bound)
    }
    is_bounded_below_on(open_interval_set(lower, upper), f)
}

/// A monotone map on `[lower, upper]` is bounded on `(lower, upper)`.
theorem monotone_on_open_interval_bounded_from_closed_interval[Domain: LinearOrder, Value: PartialOrder](
    lower: Domain, upper: Domain, f: Domain -> Value
) {
    lower <= upper and is_monotone_on(closed_interval_set(lower, upper), f)
    implies is_bounded_on(open_interval_set(lower, upper), f)
} by {
    monotone_on_open_interval_bounded_above_from_closed_interval(lower, upper, f)
    monotone_on_open_interval_bounded_below_from_closed_interval(lower, upper, f)
    is_bounded_above_on(open_interval_set(lower, upper), f)
    is_bounded_below_on(open_interval_set(lower, upper), f)
    is_bounded_on(open_interval_set(lower, upper), f)
}

/// A monotone map on `[lower, upper]` is bounded above on `(lower, upper]` by its value at `upper`.
theorem monotone_on_left_open_interval_upper_bound_from_closed_interval[Domain: LinearOrder, Value: PartialOrder](
    lower: Domain, upper: Domain, f: Domain -> Value
) {
    lower <= upper and is_monotone_on(closed_interval_set(lower, upper), f)
    implies is_upper_bound_on(left_open_interval_set(lower, upper), f, f(upper))
} by {
    monotone_on_closed_interval_upper_bound(lower, upper, f)
    is_upper_bound_on(closed_interval_set(lower, upper), f, f(upper))
    left_open_interval_set_subset_closed_interval_set(lower, upper)
    upper_bound_on_subset(left_open_interval_set(lower, upper), closed_interval_set(lower, upper), f, f(upper))
    is_upper_bound_on(left_open_interval_set(lower, upper), f, f(upper))
}

/// A monotone map on `[lower, upper]` is bounded below on `(lower, upper]` by its value at `lower`.
theorem monotone_on_left_open_interval_lower_bound_from_closed_interval[Domain: LinearOrder, Value: PartialOrder](
    lower: Domain, upper: Domain, f: Domain -> Value
) {
    lower <= upper and is_monotone_on(closed_interval_set(lower, upper), f)
    implies is_lower_bound_on(left_open_interval_set(lower, upper), f, f(lower))
} by {
    monotone_on_closed_interval_lower_bound(lower, upper, f)
    is_lower_bound_on(closed_interval_set(lower, upper), f, f(lower))
    left_open_interval_set_subset_closed_interval_set(lower, upper)
    lower_bound_on_subset(left_open_interval_set(lower, upper), closed_interval_set(lower, upper), f, f(lower))
    is_lower_bound_on(left_open_interval_set(lower, upper), f, f(lower))
}

/// A monotone map on `[lower, upper]` is bounded above on `(lower, upper]`.
theorem monotone_on_left_open_interval_bounded_above_from_closed_interval[Domain: LinearOrder, Value: PartialOrder](
    lower: Domain, upper: Domain, f: Domain -> Value
) {
    lower <= upper and is_monotone_on(closed_interval_set(lower, upper), f)
    implies is_bounded_above_on(left_open_interval_set(lower, upper), f)
} by {
    monotone_on_left_open_interval_upper_bound_from_closed_interval(lower, upper, f)
    is_upper_bound_on(left_open_interval_set(lower, upper), f, f(upper))
    exists(bound: Value) {
        is_upper_bound_on(left_open_interval_set(lower, upper), f, bound)
    }
    is_bounded_above_on(left_open_interval_set(lower, upper), f)
}

/// A monotone map on `[lower, upper]` is bounded below on `(lower, upper]`.
theorem monotone_on_left_open_interval_bounded_below_from_closed_interval[Domain: LinearOrder, Value: PartialOrder](
    lower: Domain, upper: Domain, f: Domain -> Value
) {
    lower <= upper and is_monotone_on(closed_interval_set(lower, upper), f)
    implies is_bounded_below_on(left_open_interval_set(lower, upper), f)
} by {
    monotone_on_left_open_interval_lower_bound_from_closed_interval(lower, upper, f)
    is_lower_bound_on(left_open_interval_set(lower, upper), f, f(lower))
    exists(bound: Value) {
        is_lower_bound_on(left_open_interval_set(lower, upper), f, bound)
    }
    is_bounded_below_on(left_open_interval_set(lower, upper), f)
}

/// A monotone map on `[lower, upper]` is bounded on `(lower, upper]`.
theorem monotone_on_left_open_interval_bounded_from_closed_interval[Domain: LinearOrder, Value: PartialOrder](
    lower: Domain, upper: Domain, f: Domain -> Value
) {
    lower <= upper and is_monotone_on(closed_interval_set(lower, upper), f)
    implies is_bounded_on(left_open_interval_set(lower, upper), f)
} by {
    monotone_on_left_open_interval_bounded_above_from_closed_interval(lower, upper, f)
    monotone_on_left_open_interval_bounded_below_from_closed_interval(lower, upper, f)
    is_bounded_above_on(left_open_interval_set(lower, upper), f)
    is_bounded_below_on(left_open_interval_set(lower, upper), f)
    is_bounded_on(left_open_interval_set(lower, upper), f)
}

/// A monotone map on `[lower, upper]` is bounded above on `[lower, upper)` by its value at `upper`.
theorem monotone_on_right_open_interval_upper_bound_from_closed_interval[Domain: LinearOrder, Value: PartialOrder](
    lower: Domain, upper: Domain, f: Domain -> Value
) {
    lower <= upper and is_monotone_on(closed_interval_set(lower, upper), f)
    implies is_upper_bound_on(right_open_interval_set(lower, upper), f, f(upper))
} by {
    monotone_on_closed_interval_upper_bound(lower, upper, f)
    is_upper_bound_on(closed_interval_set(lower, upper), f, f(upper))
    right_open_interval_set_subset_closed_interval_set(lower, upper)
    upper_bound_on_subset(right_open_interval_set(lower, upper), closed_interval_set(lower, upper), f, f(upper))
    is_upper_bound_on(right_open_interval_set(lower, upper), f, f(upper))
}

/// A monotone map on `[lower, upper]` is bounded below on `[lower, upper)` by its value at `lower`.
theorem monotone_on_right_open_interval_lower_bound_from_closed_interval[Domain: LinearOrder, Value: PartialOrder](
    lower: Domain, upper: Domain, f: Domain -> Value
) {
    lower <= upper and is_monotone_on(closed_interval_set(lower, upper), f)
    implies is_lower_bound_on(right_open_interval_set(lower, upper), f, f(lower))
} by {
    monotone_on_closed_interval_lower_bound(lower, upper, f)
    is_lower_bound_on(closed_interval_set(lower, upper), f, f(lower))
    right_open_interval_set_subset_closed_interval_set(lower, upper)
    lower_bound_on_subset(right_open_interval_set(lower, upper), closed_interval_set(lower, upper), f, f(lower))
    is_lower_bound_on(right_open_interval_set(lower, upper), f, f(lower))
}

/// A monotone map on `[lower, upper]` is bounded above on `[lower, upper)`.
theorem monotone_on_right_open_interval_bounded_above_from_closed_interval[Domain: LinearOrder, Value: PartialOrder](
    lower: Domain, upper: Domain, f: Domain -> Value
) {
    lower <= upper and is_monotone_on(closed_interval_set(lower, upper), f)
    implies is_bounded_above_on(right_open_interval_set(lower, upper), f)
} by {
    monotone_on_right_open_interval_upper_bound_from_closed_interval(lower, upper, f)
    is_upper_bound_on(right_open_interval_set(lower, upper), f, f(upper))
    exists(bound: Value) {
        is_upper_bound_on(right_open_interval_set(lower, upper), f, bound)
    }
    is_bounded_above_on(right_open_interval_set(lower, upper), f)
}

/// A monotone map on `[lower, upper]` is bounded below on `[lower, upper)`.
theorem monotone_on_right_open_interval_bounded_below_from_closed_interval[Domain: LinearOrder, Value: PartialOrder](
    lower: Domain, upper: Domain, f: Domain -> Value
) {
    lower <= upper and is_monotone_on(closed_interval_set(lower, upper), f)
    implies is_bounded_below_on(right_open_interval_set(lower, upper), f)
} by {
    monotone_on_right_open_interval_lower_bound_from_closed_interval(lower, upper, f)
    is_lower_bound_on(right_open_interval_set(lower, upper), f, f(lower))
    exists(bound: Value) {
        is_lower_bound_on(right_open_interval_set(lower, upper), f, bound)
    }
    is_bounded_below_on(right_open_interval_set(lower, upper), f)
}

/// A monotone map on `[lower, upper]` is bounded on `[lower, upper)`.
theorem monotone_on_right_open_interval_bounded_from_closed_interval[Domain: LinearOrder, Value: PartialOrder](
    lower: Domain, upper: Domain, f: Domain -> Value
) {
    lower <= upper and is_monotone_on(closed_interval_set(lower, upper), f)
    implies is_bounded_on(right_open_interval_set(lower, upper), f)
} by {
    monotone_on_right_open_interval_bounded_above_from_closed_interval(lower, upper, f)
    monotone_on_right_open_interval_bounded_below_from_closed_interval(lower, upper, f)
    is_bounded_above_on(right_open_interval_set(lower, upper), f)
    is_bounded_below_on(right_open_interval_set(lower, upper), f)
    is_bounded_on(right_open_interval_set(lower, upper), f)
}

/// An antitone map on `[lower, upper]` is bounded above by its value at `lower`.
theorem antitone_on_closed_interval_upper_bound[Domain: LinearOrder, Value: PartialOrder](
    lower: Domain, upper: Domain, f: Domain -> Value
) {
    lower <= upper and is_antitone_on(closed_interval_set(lower, upper), f)
    implies is_upper_bound_on(closed_interval_set(lower, upper), f, f(lower))
} by {
    if lower <= upper and is_antitone_on(closed_interval_set(lower, upper), f) {
        closed_interval_set_contains_lower(lower, upper)
        closed_interval_set(lower, upper).contains(lower)
        forall(point: Domain) {
            if closed_interval_set(lower, upper).contains(point) {
                closed_interval_set_lower_le(lower, upper, point)
                lower <= point
                antitone_on_apply(closed_interval_set(lower, upper), f, lower, point)
                f(point) <= f(lower)
            }
        }
        is_upper_bound_on(closed_interval_set(lower, upper), f, f(lower))
    }
}

/// An antitone map on `[lower, upper]` is bounded below by its value at `upper`.
theorem antitone_on_closed_interval_lower_bound[Domain: LinearOrder, Value: PartialOrder](
    lower: Domain, upper: Domain, f: Domain -> Value
) {
    lower <= upper and is_antitone_on(closed_interval_set(lower, upper), f)
    implies is_lower_bound_on(closed_interval_set(lower, upper), f, f(upper))
} by {
    if lower <= upper and is_antitone_on(closed_interval_set(lower, upper), f) {
        closed_interval_set_contains_upper(lower, upper)
        closed_interval_set(lower, upper).contains(upper)
        forall(point: Domain) {
            if closed_interval_set(lower, upper).contains(point) {
                closed_interval_set_le_upper(lower, upper, point)
                point <= upper
                antitone_on_apply(closed_interval_set(lower, upper), f, point, upper)
                f(upper) <= f(point)
            }
        }
        is_lower_bound_on(closed_interval_set(lower, upper), f, f(upper))
    }
}

/// An antitone map on `[lower, upper]` is bounded above.
theorem antitone_on_closed_interval_bounded_above[Domain: LinearOrder, Value: PartialOrder](
    lower: Domain, upper: Domain, f: Domain -> Value
) {
    lower <= upper and is_antitone_on(closed_interval_set(lower, upper), f)
    implies is_bounded_above_on(closed_interval_set(lower, upper), f)
} by {
    if lower <= upper and is_antitone_on(closed_interval_set(lower, upper), f) {
        antitone_on_closed_interval_upper_bound(lower, upper, f)
        is_upper_bound_on(closed_interval_set(lower, upper), f, f(lower))
        exists(bound: Value) {
            is_upper_bound_on(closed_interval_set(lower, upper), f, bound)
        }
        is_bounded_above_on(closed_interval_set(lower, upper), f)
    }
}

/// An antitone map on `[lower, upper]` is bounded below.
theorem antitone_on_closed_interval_bounded_below[Domain: LinearOrder, Value: PartialOrder](
    lower: Domain, upper: Domain, f: Domain -> Value
) {
    lower <= upper and is_antitone_on(closed_interval_set(lower, upper), f)
    implies is_bounded_below_on(closed_interval_set(lower, upper), f)
} by {
    if lower <= upper and is_antitone_on(closed_interval_set(lower, upper), f) {
        antitone_on_closed_interval_lower_bound(lower, upper, f)
        is_lower_bound_on(closed_interval_set(lower, upper), f, f(upper))
        exists(bound: Value) {
            is_lower_bound_on(closed_interval_set(lower, upper), f, bound)
        }
        is_bounded_below_on(closed_interval_set(lower, upper), f)
    }
}

/// An antitone map on `[lower, upper]` is bounded.
theorem antitone_on_closed_interval_bounded[Domain: LinearOrder, Value: PartialOrder](
    lower: Domain, upper: Domain, f: Domain -> Value
) {
    lower <= upper and is_antitone_on(closed_interval_set(lower, upper), f)
    implies is_bounded_on(closed_interval_set(lower, upper), f)
} by {
    if lower <= upper and is_antitone_on(closed_interval_set(lower, upper), f) {
        antitone_on_closed_interval_bounded_above(lower, upper, f)
        antitone_on_closed_interval_bounded_below(lower, upper, f)
        is_bounded_above_on(closed_interval_set(lower, upper), f)
        is_bounded_below_on(closed_interval_set(lower, upper), f)
        is_bounded_on(closed_interval_set(lower, upper), f)
    }
}

/// An antitone map on `[lower, upper]` is bounded above on `(lower, upper)` by its value at `lower`.
theorem antitone_on_open_interval_upper_bound_from_closed_interval[Domain: LinearOrder, Value: PartialOrder](
    lower: Domain, upper: Domain, f: Domain -> Value
) {
    lower <= upper and is_antitone_on(closed_interval_set(lower, upper), f)
    implies is_upper_bound_on(open_interval_set(lower, upper), f, f(lower))
} by {
    antitone_on_closed_interval_upper_bound(lower, upper, f)
    is_upper_bound_on(closed_interval_set(lower, upper), f, f(lower))
    open_interval_set_subset_closed_interval_set(lower, upper)
    upper_bound_on_subset(open_interval_set(lower, upper), closed_interval_set(lower, upper), f, f(lower))
    is_upper_bound_on(open_interval_set(lower, upper), f, f(lower))
}

/// An antitone map on `[lower, upper]` is bounded below on `(lower, upper)` by its value at `upper`.
theorem antitone_on_open_interval_lower_bound_from_closed_interval[Domain: LinearOrder, Value: PartialOrder](
    lower: Domain, upper: Domain, f: Domain -> Value
) {
    lower <= upper and is_antitone_on(closed_interval_set(lower, upper), f)
    implies is_lower_bound_on(open_interval_set(lower, upper), f, f(upper))
} by {
    antitone_on_closed_interval_lower_bound(lower, upper, f)
    is_lower_bound_on(closed_interval_set(lower, upper), f, f(upper))
    open_interval_set_subset_closed_interval_set(lower, upper)
    lower_bound_on_subset(open_interval_set(lower, upper), closed_interval_set(lower, upper), f, f(upper))
    is_lower_bound_on(open_interval_set(lower, upper), f, f(upper))
}

/// An antitone map on `[lower, upper]` is bounded above on `(lower, upper)`.
theorem antitone_on_open_interval_bounded_above_from_closed_interval[Domain: LinearOrder, Value: PartialOrder](
    lower: Domain, upper: Domain, f: Domain -> Value
) {
    lower <= upper and is_antitone_on(closed_interval_set(lower, upper), f)
    implies is_bounded_above_on(open_interval_set(lower, upper), f)
} by {
    antitone_on_open_interval_upper_bound_from_closed_interval(lower, upper, f)
    is_upper_bound_on(open_interval_set(lower, upper), f, f(lower))
    exists(bound: Value) {
        is_upper_bound_on(open_interval_set(lower, upper), f, bound)
    }
    is_bounded_above_on(open_interval_set(lower, upper), f)
}

/// An antitone map on `[lower, upper]` is bounded below on `(lower, upper)`.
theorem antitone_on_open_interval_bounded_below_from_closed_interval[Domain: LinearOrder, Value: PartialOrder](
    lower: Domain, upper: Domain, f: Domain -> Value
) {
    lower <= upper and is_antitone_on(closed_interval_set(lower, upper), f)
    implies is_bounded_below_on(open_interval_set(lower, upper), f)
} by {
    antitone_on_open_interval_lower_bound_from_closed_interval(lower, upper, f)
    is_lower_bound_on(open_interval_set(lower, upper), f, f(upper))
    exists(bound: Value) {
        is_lower_bound_on(open_interval_set(lower, upper), f, bound)
    }
    is_bounded_below_on(open_interval_set(lower, upper), f)
}

/// An antitone map on `[lower, upper]` is bounded on `(lower, upper)`.
theorem antitone_on_open_interval_bounded_from_closed_interval[Domain: LinearOrder, Value: PartialOrder](
    lower: Domain, upper: Domain, f: Domain -> Value
) {
    lower <= upper and is_antitone_on(closed_interval_set(lower, upper), f)
    implies is_bounded_on(open_interval_set(lower, upper), f)
} by {
    antitone_on_open_interval_bounded_above_from_closed_interval(lower, upper, f)
    antitone_on_open_interval_bounded_below_from_closed_interval(lower, upper, f)
    is_bounded_above_on(open_interval_set(lower, upper), f)
    is_bounded_below_on(open_interval_set(lower, upper), f)
    is_bounded_on(open_interval_set(lower, upper), f)
}

/// An antitone map on `[lower, upper]` is bounded above on `(lower, upper]` by its value at `lower`.
theorem antitone_on_left_open_interval_upper_bound_from_closed_interval[Domain: LinearOrder, Value: PartialOrder](
    lower: Domain, upper: Domain, f: Domain -> Value
) {
    lower <= upper and is_antitone_on(closed_interval_set(lower, upper), f)
    implies is_upper_bound_on(left_open_interval_set(lower, upper), f, f(lower))
} by {
    antitone_on_closed_interval_upper_bound(lower, upper, f)
    is_upper_bound_on(closed_interval_set(lower, upper), f, f(lower))
    left_open_interval_set_subset_closed_interval_set(lower, upper)
    upper_bound_on_subset(left_open_interval_set(lower, upper), closed_interval_set(lower, upper), f, f(lower))
    is_upper_bound_on(left_open_interval_set(lower, upper), f, f(lower))
}

/// An antitone map on `[lower, upper]` is bounded below on `(lower, upper]` by its value at `upper`.
theorem antitone_on_left_open_interval_lower_bound_from_closed_interval[Domain: LinearOrder, Value: PartialOrder](
    lower: Domain, upper: Domain, f: Domain -> Value
) {
    lower <= upper and is_antitone_on(closed_interval_set(lower, upper), f)
    implies is_lower_bound_on(left_open_interval_set(lower, upper), f, f(upper))
} by {
    antitone_on_closed_interval_lower_bound(lower, upper, f)
    is_lower_bound_on(closed_interval_set(lower, upper), f, f(upper))
    left_open_interval_set_subset_closed_interval_set(lower, upper)
    lower_bound_on_subset(left_open_interval_set(lower, upper), closed_interval_set(lower, upper), f, f(upper))
    is_lower_bound_on(left_open_interval_set(lower, upper), f, f(upper))
}

/// An antitone map on `[lower, upper]` is bounded above on `(lower, upper]`.
theorem antitone_on_left_open_interval_bounded_above_from_closed_interval[Domain: LinearOrder, Value: PartialOrder](
    lower: Domain, upper: Domain, f: Domain -> Value
) {
    lower <= upper and is_antitone_on(closed_interval_set(lower, upper), f)
    implies is_bounded_above_on(left_open_interval_set(lower, upper), f)
} by {
    antitone_on_left_open_interval_upper_bound_from_closed_interval(lower, upper, f)
    is_upper_bound_on(left_open_interval_set(lower, upper), f, f(lower))
    exists(bound: Value) {
        is_upper_bound_on(left_open_interval_set(lower, upper), f, bound)
    }
    is_bounded_above_on(left_open_interval_set(lower, upper), f)
}

/// An antitone map on `[lower, upper]` is bounded below on `(lower, upper]`.
theorem antitone_on_left_open_interval_bounded_below_from_closed_interval[Domain: LinearOrder, Value: PartialOrder](
    lower: Domain, upper: Domain, f: Domain -> Value
) {
    lower <= upper and is_antitone_on(closed_interval_set(lower, upper), f)
    implies is_bounded_below_on(left_open_interval_set(lower, upper), f)
} by {
    antitone_on_left_open_interval_lower_bound_from_closed_interval(lower, upper, f)
    is_lower_bound_on(left_open_interval_set(lower, upper), f, f(upper))
    exists(bound: Value) {
        is_lower_bound_on(left_open_interval_set(lower, upper), f, bound)
    }
    is_bounded_below_on(left_open_interval_set(lower, upper), f)
}

/// An antitone map on `[lower, upper]` is bounded on `(lower, upper]`.
theorem antitone_on_left_open_interval_bounded_from_closed_interval[Domain: LinearOrder, Value: PartialOrder](
    lower: Domain, upper: Domain, f: Domain -> Value
) {
    lower <= upper and is_antitone_on(closed_interval_set(lower, upper), f)
    implies is_bounded_on(left_open_interval_set(lower, upper), f)
} by {
    antitone_on_left_open_interval_bounded_above_from_closed_interval(lower, upper, f)
    antitone_on_left_open_interval_bounded_below_from_closed_interval(lower, upper, f)
    is_bounded_above_on(left_open_interval_set(lower, upper), f)
    is_bounded_below_on(left_open_interval_set(lower, upper), f)
    is_bounded_on(left_open_interval_set(lower, upper), f)
}

/// An antitone map on `[lower, upper]` is bounded above on `[lower, upper)` by its value at `lower`.
theorem antitone_on_right_open_interval_upper_bound_from_closed_interval[Domain: LinearOrder, Value: PartialOrder](
    lower: Domain, upper: Domain, f: Domain -> Value
) {
    lower <= upper and is_antitone_on(closed_interval_set(lower, upper), f)
    implies is_upper_bound_on(right_open_interval_set(lower, upper), f, f(lower))
} by {
    antitone_on_closed_interval_upper_bound(lower, upper, f)
    is_upper_bound_on(closed_interval_set(lower, upper), f, f(lower))
    right_open_interval_set_subset_closed_interval_set(lower, upper)
    upper_bound_on_subset(right_open_interval_set(lower, upper), closed_interval_set(lower, upper), f, f(lower))
    is_upper_bound_on(right_open_interval_set(lower, upper), f, f(lower))
}

/// An antitone map on `[lower, upper]` is bounded below on `[lower, upper)` by its value at `upper`.
theorem antitone_on_right_open_interval_lower_bound_from_closed_interval[Domain: LinearOrder, Value: PartialOrder](
    lower: Domain, upper: Domain, f: Domain -> Value
) {
    lower <= upper and is_antitone_on(closed_interval_set(lower, upper), f)
    implies is_lower_bound_on(right_open_interval_set(lower, upper), f, f(upper))
} by {
    antitone_on_closed_interval_lower_bound(lower, upper, f)
    is_lower_bound_on(closed_interval_set(lower, upper), f, f(upper))
    right_open_interval_set_subset_closed_interval_set(lower, upper)
    lower_bound_on_subset(right_open_interval_set(lower, upper), closed_interval_set(lower, upper), f, f(upper))
    is_lower_bound_on(right_open_interval_set(lower, upper), f, f(upper))
}

/// An antitone map on `[lower, upper]` is bounded above on `[lower, upper)`.
theorem antitone_on_right_open_interval_bounded_above_from_closed_interval[Domain: LinearOrder, Value: PartialOrder](
    lower: Domain, upper: Domain, f: Domain -> Value
) {
    lower <= upper and is_antitone_on(closed_interval_set(lower, upper), f)
    implies is_bounded_above_on(right_open_interval_set(lower, upper), f)
} by {
    antitone_on_right_open_interval_upper_bound_from_closed_interval(lower, upper, f)
    is_upper_bound_on(right_open_interval_set(lower, upper), f, f(lower))
    exists(bound: Value) {
        is_upper_bound_on(right_open_interval_set(lower, upper), f, bound)
    }
    is_bounded_above_on(right_open_interval_set(lower, upper), f)
}

/// An antitone map on `[lower, upper]` is bounded below on `[lower, upper)`.
theorem antitone_on_right_open_interval_bounded_below_from_closed_interval[Domain: LinearOrder, Value: PartialOrder](
    lower: Domain, upper: Domain, f: Domain -> Value
) {
    lower <= upper and is_antitone_on(closed_interval_set(lower, upper), f)
    implies is_bounded_below_on(right_open_interval_set(lower, upper), f)
} by {
    antitone_on_right_open_interval_lower_bound_from_closed_interval(lower, upper, f)
    is_lower_bound_on(right_open_interval_set(lower, upper), f, f(upper))
    exists(bound: Value) {
        is_lower_bound_on(right_open_interval_set(lower, upper), f, bound)
    }
    is_bounded_below_on(right_open_interval_set(lower, upper), f)
}

/// An antitone map on `[lower, upper]` is bounded on `[lower, upper)`.
theorem antitone_on_right_open_interval_bounded_from_closed_interval[Domain: LinearOrder, Value: PartialOrder](
    lower: Domain, upper: Domain, f: Domain -> Value
) {
    lower <= upper and is_antitone_on(closed_interval_set(lower, upper), f)
    implies is_bounded_on(right_open_interval_set(lower, upper), f)
} by {
    antitone_on_right_open_interval_bounded_above_from_closed_interval(lower, upper, f)
    antitone_on_right_open_interval_bounded_below_from_closed_interval(lower, upper, f)
    is_bounded_above_on(right_open_interval_set(lower, upper), f)
    is_bounded_below_on(right_open_interval_set(lower, upper), f)
    is_bounded_on(right_open_interval_set(lower, upper), f)
}

/// A monotone map on `[lower, upper]` attains its maximum at `upper`.
theorem monotone_on_closed_interval_attains_maximum_at_upper[Domain: LinearOrder, Value: PartialOrder](
    lower: Domain, upper: Domain, f: Domain -> Value
) {
    lower <= upper and is_monotone_on(closed_interval_set(lower, upper), f)
    implies attains_maximum_at(closed_interval_set(lower, upper), f, upper)
} by {
    closed_interval_set_contains_upper(lower, upper)
    monotone_on_closed_interval_upper_bound(lower, upper, f)
    closed_interval_set(lower, upper).contains(upper)
    is_upper_bound_on(closed_interval_set(lower, upper), f, f(upper))
    attains_maximum_at(closed_interval_set(lower, upper), f, upper)
}

/// A monotone map on `[lower, upper]` attains its minimum at `lower`.
theorem monotone_on_closed_interval_attains_minimum_at_lower[Domain: LinearOrder, Value: PartialOrder](
    lower: Domain, upper: Domain, f: Domain -> Value
) {
    lower <= upper and is_monotone_on(closed_interval_set(lower, upper), f)
    implies attains_minimum_at(closed_interval_set(lower, upper), f, lower)
} by {
    closed_interval_set_contains_lower(lower, upper)
    monotone_on_closed_interval_lower_bound(lower, upper, f)
    closed_interval_set(lower, upper).contains(lower)
    is_lower_bound_on(closed_interval_set(lower, upper), f, f(lower))
    attains_minimum_at(closed_interval_set(lower, upper), f, lower)
}

/// A monotone map on `[lower, upper]` attains a maximum on the interval.
theorem monotone_on_closed_interval_attains_maximum[Domain: LinearOrder, Value: PartialOrder](
    lower: Domain, upper: Domain, f: Domain -> Value
) {
    lower <= upper and is_monotone_on(closed_interval_set(lower, upper), f)
    implies attains_maximum_on(closed_interval_set(lower, upper), f)
} by {
    monotone_on_closed_interval_attains_maximum_at_upper(lower, upper, f)
    attains_maximum_at(closed_interval_set(lower, upper), f, upper)
    exists(point: Domain) {
        attains_maximum_at(closed_interval_set(lower, upper), f, point)
    }
    attains_maximum_on(closed_interval_set(lower, upper), f)
}

/// A monotone map on `[lower, upper]` attains a minimum on the interval.
theorem monotone_on_closed_interval_attains_minimum[Domain: LinearOrder, Value: PartialOrder](
    lower: Domain, upper: Domain, f: Domain -> Value
) {
    lower <= upper and is_monotone_on(closed_interval_set(lower, upper), f)
    implies attains_minimum_on(closed_interval_set(lower, upper), f)
} by {
    monotone_on_closed_interval_attains_minimum_at_lower(lower, upper, f)
    attains_minimum_at(closed_interval_set(lower, upper), f, lower)
    exists(point: Domain) {
        attains_minimum_at(closed_interval_set(lower, upper), f, point)
    }
    attains_minimum_on(closed_interval_set(lower, upper), f)
}

/// An antitone map on `[lower, upper]` attains its maximum at `lower`.
theorem antitone_on_closed_interval_attains_maximum_at_lower[Domain: LinearOrder, Value: PartialOrder](
    lower: Domain, upper: Domain, f: Domain -> Value
) {
    lower <= upper and is_antitone_on(closed_interval_set(lower, upper), f)
    implies attains_maximum_at(closed_interval_set(lower, upper), f, lower)
} by {
    closed_interval_set_contains_lower(lower, upper)
    antitone_on_closed_interval_upper_bound(lower, upper, f)
    closed_interval_set(lower, upper).contains(lower)
    is_upper_bound_on(closed_interval_set(lower, upper), f, f(lower))
    attains_maximum_at(closed_interval_set(lower, upper), f, lower)
}

/// An antitone map on `[lower, upper]` attains its minimum at `upper`.
theorem antitone_on_closed_interval_attains_minimum_at_upper[Domain: LinearOrder, Value: PartialOrder](
    lower: Domain, upper: Domain, f: Domain -> Value
) {
    lower <= upper and is_antitone_on(closed_interval_set(lower, upper), f)
    implies attains_minimum_at(closed_interval_set(lower, upper), f, upper)
} by {
    closed_interval_set_contains_upper(lower, upper)
    antitone_on_closed_interval_lower_bound(lower, upper, f)
    closed_interval_set(lower, upper).contains(upper)
    is_lower_bound_on(closed_interval_set(lower, upper), f, f(upper))
    attains_minimum_at(closed_interval_set(lower, upper), f, upper)
}

/// An antitone map on `[lower, upper]` attains a maximum on the interval.
theorem antitone_on_closed_interval_attains_maximum[Domain: LinearOrder, Value: PartialOrder](
    lower: Domain, upper: Domain, f: Domain -> Value
) {
    lower <= upper and is_antitone_on(closed_interval_set(lower, upper), f)
    implies attains_maximum_on(closed_interval_set(lower, upper), f)
} by {
    antitone_on_closed_interval_attains_maximum_at_lower(lower, upper, f)
    attains_maximum_at(closed_interval_set(lower, upper), f, lower)
    exists(point: Domain) {
        attains_maximum_at(closed_interval_set(lower, upper), f, point)
    }
    attains_maximum_on(closed_interval_set(lower, upper), f)
}

/// An antitone map on `[lower, upper]` attains a minimum on the interval.
theorem antitone_on_closed_interval_attains_minimum[Domain: LinearOrder, Value: PartialOrder](
    lower: Domain, upper: Domain, f: Domain -> Value
) {
    lower <= upper and is_antitone_on(closed_interval_set(lower, upper), f)
    implies attains_minimum_on(closed_interval_set(lower, upper), f)
} by {
    antitone_on_closed_interval_attains_minimum_at_upper(lower, upper, f)
    attains_minimum_at(closed_interval_set(lower, upper), f, upper)
    exists(point: Domain) {
        attains_minimum_at(closed_interval_set(lower, upper), f, point)
    }
    attains_minimum_on(closed_interval_set(lower, upper), f)
}

/// A monotone map on `(lower, upper]` is bounded above by its value at `upper`.
theorem monotone_on_left_open_interval_upper_bound[Domain: LinearOrder, Value: PartialOrder](
    lower: Domain, upper: Domain, f: Domain -> Value
) {
    lower < upper and is_monotone_on(left_open_interval_set(lower, upper), f)
    implies is_upper_bound_on(left_open_interval_set(lower, upper), f, f(upper))
} by {
    left_open_interval_set_contains_upper(lower, upper)
    left_open_interval_set(lower, upper).contains(upper)
    forall(point: Domain) {
        if left_open_interval_set(lower, upper).contains(point) {
            left_open_interval_set_le_upper(lower, upper, point)
            point <= upper
            monotone_on_apply(left_open_interval_set(lower, upper), f, point, upper)
            f(point) <= f(upper)
        }
    }
    is_upper_bound_on(left_open_interval_set(lower, upper), f, f(upper))
}

/// A monotone map on `(lower, upper]` attains its maximum at `upper`.
theorem monotone_on_left_open_interval_attains_maximum_at_upper[Domain: LinearOrder, Value: PartialOrder](
    lower: Domain, upper: Domain, f: Domain -> Value
) {
    lower < upper and is_monotone_on(left_open_interval_set(lower, upper), f)
    implies attains_maximum_at(left_open_interval_set(lower, upper), f, upper)
} by {
    left_open_interval_set_contains_upper(lower, upper)
    monotone_on_left_open_interval_upper_bound(lower, upper, f)
    left_open_interval_set(lower, upper).contains(upper)
    is_upper_bound_on(left_open_interval_set(lower, upper), f, f(upper))
    attains_maximum_at(left_open_interval_set(lower, upper), f, upper)
}

/// A monotone map on `(lower, upper]` attains a maximum on the interval.
theorem monotone_on_left_open_interval_attains_maximum[Domain: LinearOrder, Value: PartialOrder](
    lower: Domain, upper: Domain, f: Domain -> Value
) {
    lower < upper and is_monotone_on(left_open_interval_set(lower, upper), f)
    implies attains_maximum_on(left_open_interval_set(lower, upper), f)
} by {
    monotone_on_left_open_interval_attains_maximum_at_upper(lower, upper, f)
    attains_maximum_at(left_open_interval_set(lower, upper), f, upper)
    exists(point: Domain) {
        attains_maximum_at(left_open_interval_set(lower, upper), f, point)
    }
    attains_maximum_on(left_open_interval_set(lower, upper), f)
}

/// A monotone map on `(lower, upper]` is bounded above.
theorem monotone_on_left_open_interval_bounded_above[Domain: LinearOrder, Value: PartialOrder](
    lower: Domain, upper: Domain, f: Domain -> Value
) {
    lower < upper and is_monotone_on(left_open_interval_set(lower, upper), f)
    implies is_bounded_above_on(left_open_interval_set(lower, upper), f)
} by {
    monotone_on_left_open_interval_attains_maximum(lower, upper, f)
    attains_maximum_imp_bounded_above(left_open_interval_set(lower, upper), f)
    is_bounded_above_on(left_open_interval_set(lower, upper), f)
}

/// A monotone map on `[lower, upper)` is bounded below by its value at `lower`.
theorem monotone_on_right_open_interval_lower_bound[Domain: LinearOrder, Value: PartialOrder](
    lower: Domain, upper: Domain, f: Domain -> Value
) {
    lower < upper and is_monotone_on(right_open_interval_set(lower, upper), f)
    implies is_lower_bound_on(right_open_interval_set(lower, upper), f, f(lower))
} by {
    right_open_interval_set_contains_lower(lower, upper)
    right_open_interval_set(lower, upper).contains(lower)
    forall(point: Domain) {
        if right_open_interval_set(lower, upper).contains(point) {
            right_open_interval_set_lower_le(lower, upper, point)
            lower <= point
            monotone_on_apply(right_open_interval_set(lower, upper), f, lower, point)
            f(lower) <= f(point)
        }
    }
    is_lower_bound_on(right_open_interval_set(lower, upper), f, f(lower))
}

/// A monotone map on `[lower, upper)` attains its minimum at `lower`.
theorem monotone_on_right_open_interval_attains_minimum_at_lower[Domain: LinearOrder, Value: PartialOrder](
    lower: Domain, upper: Domain, f: Domain -> Value
) {
    lower < upper and is_monotone_on(right_open_interval_set(lower, upper), f)
    implies attains_minimum_at(right_open_interval_set(lower, upper), f, lower)
} by {
    right_open_interval_set_contains_lower(lower, upper)
    monotone_on_right_open_interval_lower_bound(lower, upper, f)
    right_open_interval_set(lower, upper).contains(lower)
    is_lower_bound_on(right_open_interval_set(lower, upper), f, f(lower))
    attains_minimum_at(right_open_interval_set(lower, upper), f, lower)
}

/// A monotone map on `[lower, upper)` attains a minimum on the interval.
theorem monotone_on_right_open_interval_attains_minimum[Domain: LinearOrder, Value: PartialOrder](
    lower: Domain, upper: Domain, f: Domain -> Value
) {
    lower < upper and is_monotone_on(right_open_interval_set(lower, upper), f)
    implies attains_minimum_on(right_open_interval_set(lower, upper), f)
} by {
    monotone_on_right_open_interval_attains_minimum_at_lower(lower, upper, f)
    attains_minimum_at(right_open_interval_set(lower, upper), f, lower)
    exists(point: Domain) {
        attains_minimum_at(right_open_interval_set(lower, upper), f, point)
    }
    attains_minimum_on(right_open_interval_set(lower, upper), f)
}

/// A monotone map on `[lower, upper)` is bounded below.
theorem monotone_on_right_open_interval_bounded_below[Domain: LinearOrder, Value: PartialOrder](
    lower: Domain, upper: Domain, f: Domain -> Value
) {
    lower < upper and is_monotone_on(right_open_interval_set(lower, upper), f)
    implies is_bounded_below_on(right_open_interval_set(lower, upper), f)
} by {
    monotone_on_right_open_interval_attains_minimum(lower, upper, f)
    attains_minimum_imp_bounded_below(right_open_interval_set(lower, upper), f)
    is_bounded_below_on(right_open_interval_set(lower, upper), f)
}

/// An antitone map on `(lower, upper]` is bounded below by its value at `upper`.
theorem antitone_on_left_open_interval_lower_bound[Domain: LinearOrder, Value: PartialOrder](
    lower: Domain, upper: Domain, f: Domain -> Value
) {
    lower < upper and is_antitone_on(left_open_interval_set(lower, upper), f)
    implies is_lower_bound_on(left_open_interval_set(lower, upper), f, f(upper))
} by {
    left_open_interval_set_contains_upper(lower, upper)
    left_open_interval_set(lower, upper).contains(upper)
    forall(point: Domain) {
        if left_open_interval_set(lower, upper).contains(point) {
            left_open_interval_set_le_upper(lower, upper, point)
            point <= upper
            antitone_on_apply(left_open_interval_set(lower, upper), f, point, upper)
            f(upper) <= f(point)
        }
    }
    is_lower_bound_on(left_open_interval_set(lower, upper), f, f(upper))
}

/// An antitone map on `(lower, upper]` attains its minimum at `upper`.
theorem antitone_on_left_open_interval_attains_minimum_at_upper[Domain: LinearOrder, Value: PartialOrder](
    lower: Domain, upper: Domain, f: Domain -> Value
) {
    lower < upper and is_antitone_on(left_open_interval_set(lower, upper), f)
    implies attains_minimum_at(left_open_interval_set(lower, upper), f, upper)
} by {
    left_open_interval_set_contains_upper(lower, upper)
    antitone_on_left_open_interval_lower_bound(lower, upper, f)
    left_open_interval_set(lower, upper).contains(upper)
    is_lower_bound_on(left_open_interval_set(lower, upper), f, f(upper))
    attains_minimum_at(left_open_interval_set(lower, upper), f, upper)
}

/// An antitone map on `(lower, upper]` attains a minimum on the interval.
theorem antitone_on_left_open_interval_attains_minimum[Domain: LinearOrder, Value: PartialOrder](
    lower: Domain, upper: Domain, f: Domain -> Value
) {
    lower < upper and is_antitone_on(left_open_interval_set(lower, upper), f)
    implies attains_minimum_on(left_open_interval_set(lower, upper), f)
} by {
    antitone_on_left_open_interval_attains_minimum_at_upper(lower, upper, f)
    attains_minimum_at(left_open_interval_set(lower, upper), f, upper)
    exists(point: Domain) {
        attains_minimum_at(left_open_interval_set(lower, upper), f, point)
    }
    attains_minimum_on(left_open_interval_set(lower, upper), f)
}

/// An antitone map on `(lower, upper]` is bounded below.
theorem antitone_on_left_open_interval_bounded_below[Domain: LinearOrder, Value: PartialOrder](
    lower: Domain, upper: Domain, f: Domain -> Value
) {
    lower < upper and is_antitone_on(left_open_interval_set(lower, upper), f)
    implies is_bounded_below_on(left_open_interval_set(lower, upper), f)
} by {
    antitone_on_left_open_interval_attains_minimum(lower, upper, f)
    attains_minimum_imp_bounded_below(left_open_interval_set(lower, upper), f)
    is_bounded_below_on(left_open_interval_set(lower, upper), f)
}

/// An antitone map on `[lower, upper)` is bounded above by its value at `lower`.
theorem antitone_on_right_open_interval_upper_bound[Domain: LinearOrder, Value: PartialOrder](
    lower: Domain, upper: Domain, f: Domain -> Value
) {
    lower < upper and is_antitone_on(right_open_interval_set(lower, upper), f)
    implies is_upper_bound_on(right_open_interval_set(lower, upper), f, f(lower))
} by {
    right_open_interval_set_contains_lower(lower, upper)
    right_open_interval_set(lower, upper).contains(lower)
    forall(point: Domain) {
        if right_open_interval_set(lower, upper).contains(point) {
            right_open_interval_set_lower_le(lower, upper, point)
            lower <= point
            antitone_on_apply(right_open_interval_set(lower, upper), f, lower, point)
            f(point) <= f(lower)
        }
    }
    is_upper_bound_on(right_open_interval_set(lower, upper), f, f(lower))
}

/// An antitone map on `[lower, upper)` attains its maximum at `lower`.
theorem antitone_on_right_open_interval_attains_maximum_at_lower[Domain: LinearOrder, Value: PartialOrder](
    lower: Domain, upper: Domain, f: Domain -> Value
) {
    lower < upper and is_antitone_on(right_open_interval_set(lower, upper), f)
    implies attains_maximum_at(right_open_interval_set(lower, upper), f, lower)
} by {
    right_open_interval_set_contains_lower(lower, upper)
    antitone_on_right_open_interval_upper_bound(lower, upper, f)
    right_open_interval_set(lower, upper).contains(lower)
    is_upper_bound_on(right_open_interval_set(lower, upper), f, f(lower))
    attains_maximum_at(right_open_interval_set(lower, upper), f, lower)
}

/// An antitone map on `[lower, upper)` attains a maximum on the interval.
theorem antitone_on_right_open_interval_attains_maximum[Domain: LinearOrder, Value: PartialOrder](
    lower: Domain, upper: Domain, f: Domain -> Value
) {
    lower < upper and is_antitone_on(right_open_interval_set(lower, upper), f)
    implies attains_maximum_on(right_open_interval_set(lower, upper), f)
} by {
    antitone_on_right_open_interval_attains_maximum_at_lower(lower, upper, f)
    attains_maximum_at(right_open_interval_set(lower, upper), f, lower)
    exists(point: Domain) {
        attains_maximum_at(right_open_interval_set(lower, upper), f, point)
    }
    attains_maximum_on(right_open_interval_set(lower, upper), f)
}

/// An antitone map on `[lower, upper)` is bounded above.
theorem antitone_on_right_open_interval_bounded_above[Domain: LinearOrder, Value: PartialOrder](
    lower: Domain, upper: Domain, f: Domain -> Value
) {
    lower < upper and is_antitone_on(right_open_interval_set(lower, upper), f)
    implies is_bounded_above_on(right_open_interval_set(lower, upper), f)
} by {
    antitone_on_right_open_interval_attains_maximum(lower, upper, f)
    attains_maximum_imp_bounded_above(right_open_interval_set(lower, upper), f)
    is_bounded_above_on(right_open_interval_set(lower, upper), f)
}
