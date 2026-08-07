/// Monotone and antitone maps restricted to a subset of their domain.

from data.basic.functions import compose, identity_fn
from order import PartialOrder, lte_refl, lt_imp_lte
from order_set.function_order_on import function_eq_on, function_eq_on_apply, function_eq_on_symm
from order import is_monotone, is_antitone, is_strict_monotone, is_strict_antitone,
    monotone_apply, antitone_apply, strict_monotone_apply, strict_antitone_apply,
    identity_is_monotone, identity_is_strict_monotone
from data.basic.set import Set, set_image, maps_into_set_image

/// True if `f` preserves the non-strict order on `domain`.
define is_monotone_on[Domain: PartialOrder, Value: PartialOrder](
    domain: Set[Domain], f: Domain -> Value
) -> Bool {
    forall(left: Domain, right: Domain) {
        domain.contains(left) and domain.contains(right) and left <= right implies f(left) <= f(right)
    }
}

/// True if `f` reverses the non-strict order on `domain`.
define is_antitone_on[Domain: PartialOrder, Value: PartialOrder](
    domain: Set[Domain], f: Domain -> Value
) -> Bool {
    forall(left: Domain, right: Domain) {
        domain.contains(left) and domain.contains(right) and left <= right implies f(right) <= f(left)
    }
}

/// True if `f` preserves strict order on `domain`.
define is_strict_monotone_on[Domain: PartialOrder, Value: PartialOrder](
    domain: Set[Domain], f: Domain -> Value
) -> Bool {
    forall(left: Domain, right: Domain) {
        domain.contains(left) and domain.contains(right) and left < right implies f(left) < f(right)
    }
}

/// True if `f` reverses strict order on `domain`.
define is_strict_antitone_on[Domain: PartialOrder, Value: PartialOrder](
    domain: Set[Domain], f: Domain -> Value
) -> Bool {
    forall(left: Domain, right: Domain) {
        domain.contains(left) and domain.contains(right) and left < right implies f(right) < f(left)
    }
}

/// A monotone map on a set carries an ordered pair of points to an ordered pair of values.
theorem monotone_on_apply[Domain: PartialOrder, Value: PartialOrder](
    domain: Set[Domain], f: Domain -> Value, left: Domain, right: Domain
) {
    is_monotone_on(domain, f) and domain.contains(left) and domain.contains(right) and left <= right
    implies f(left) <= f(right)
} by {
    if is_monotone_on(domain, f) and domain.contains(left) and domain.contains(right) and left <= right {
        is_monotone_on(domain, f) = forall(first: Domain, second: Domain) {
            domain.contains(first) and domain.contains(second) and first <= second implies f(first) <= f(second)
        }
        domain.contains(left) and domain.contains(right) and left <= right implies f(left) <= f(right)
        f(left) <= f(right)
    }
}

/// An antitone map on a set reverses an ordered pair of points.
theorem antitone_on_apply[Domain: PartialOrder, Value: PartialOrder](
    domain: Set[Domain], f: Domain -> Value, left: Domain, right: Domain
) {
    is_antitone_on(domain, f) and domain.contains(left) and domain.contains(right) and left <= right
    implies f(right) <= f(left)
} by {
    if is_antitone_on(domain, f) and domain.contains(left) and domain.contains(right) and left <= right {
        is_antitone_on(domain, f) = forall(first: Domain, second: Domain) {
            domain.contains(first) and domain.contains(second) and first <= second implies f(second) <= f(first)
        }
        domain.contains(left) and domain.contains(right) and left <= right implies f(right) <= f(left)
        f(right) <= f(left)
    }
}

/// A strictly monotone map on a set carries a strictly ordered pair to a strictly ordered pair.
theorem strict_monotone_on_apply[Domain: PartialOrder, Value: PartialOrder](
    domain: Set[Domain], f: Domain -> Value, left: Domain, right: Domain
) {
    is_strict_monotone_on(domain, f) and domain.contains(left) and domain.contains(right) and left < right
    implies f(left) < f(right)
} by {
    if is_strict_monotone_on(domain, f) and domain.contains(left) and domain.contains(right) and left < right {
        is_strict_monotone_on(domain, f) = forall(first: Domain, second: Domain) {
            domain.contains(first) and domain.contains(second) and first < second implies f(first) < f(second)
        }
        domain.contains(left) and domain.contains(right) and left < right implies f(left) < f(right)
        f(left) < f(right)
    }
}

/// A strictly antitone map on a set reverses a strictly ordered pair.
theorem strict_antitone_on_apply[Domain: PartialOrder, Value: PartialOrder](
    domain: Set[Domain], f: Domain -> Value, left: Domain, right: Domain
) {
    is_strict_antitone_on(domain, f) and domain.contains(left) and domain.contains(right) and left < right
    implies f(right) < f(left)
} by {
    if is_strict_antitone_on(domain, f) and domain.contains(left) and domain.contains(right) and left < right {
        is_strict_antitone_on(domain, f) = forall(first: Domain, second: Domain) {
            domain.contains(first) and domain.contains(second) and first < second implies f(second) < f(first)
        }
        domain.contains(left) and domain.contains(right) and left < right implies f(right) < f(left)
        f(right) < f(left)
    }
}

/// Monotonicity on a set is preserved by pointwise equality on that set.
theorem monotone_on_of_function_eq_on[Domain: PartialOrder, Value: PartialOrder](
    domain: Set[Domain], original: Domain -> Value, replacement: Domain -> Value
) {
    function_eq_on(domain, original, replacement) and is_monotone_on(domain, original)
    implies is_monotone_on(domain, replacement)
} by {
    forall(left: Domain, right: Domain) {
        if domain.contains(left) and domain.contains(right) and left <= right {
            function_eq_on_apply(domain, original, replacement, left)
            function_eq_on_apply(domain, original, replacement, right)
            monotone_on_apply(domain, original, left, right)
            original(left) = replacement(left)
            original(right) = replacement(right)
            original(left) <= original(right)
            replacement(left) <= replacement(right)
        }
    }
}

/// Antitonicity on a set is preserved by pointwise equality on that set.
theorem antitone_on_of_function_eq_on[Domain: PartialOrder, Value: PartialOrder](
    domain: Set[Domain], original: Domain -> Value, replacement: Domain -> Value
) {
    function_eq_on(domain, original, replacement) and is_antitone_on(domain, original)
    implies is_antitone_on(domain, replacement)
} by {
    forall(left: Domain, right: Domain) {
        if domain.contains(left) and domain.contains(right) and left <= right {
            function_eq_on_apply(domain, original, replacement, left)
            function_eq_on_apply(domain, original, replacement, right)
            antitone_on_apply(domain, original, left, right)
            original(left) = replacement(left)
            original(right) = replacement(right)
            original(right) <= original(left)
            replacement(right) <= replacement(left)
        }
    }
}

/// Strict monotonicity on a set is preserved by pointwise equality on that set.
theorem strict_monotone_on_of_function_eq_on[Domain: PartialOrder, Value: PartialOrder](
    domain: Set[Domain], original: Domain -> Value, replacement: Domain -> Value
) {
    function_eq_on(domain, original, replacement) and is_strict_monotone_on(domain, original)
    implies is_strict_monotone_on(domain, replacement)
} by {
    forall(left: Domain, right: Domain) {
        if domain.contains(left) and domain.contains(right) and left < right {
            function_eq_on_apply(domain, original, replacement, left)
            function_eq_on_apply(domain, original, replacement, right)
            strict_monotone_on_apply(domain, original, left, right)
            original(left) = replacement(left)
            original(right) = replacement(right)
            original(left) < original(right)
            replacement(left) < replacement(right)
        }
    }
}

/// Strict antitonicity on a set is preserved by pointwise equality on that set.
theorem strict_antitone_on_of_function_eq_on[Domain: PartialOrder, Value: PartialOrder](
    domain: Set[Domain], original: Domain -> Value, replacement: Domain -> Value
) {
    function_eq_on(domain, original, replacement) and is_strict_antitone_on(domain, original)
    implies is_strict_antitone_on(domain, replacement)
} by {
    forall(left: Domain, right: Domain) {
        if domain.contains(left) and domain.contains(right) and left < right {
            function_eq_on_apply(domain, original, replacement, left)
            function_eq_on_apply(domain, original, replacement, right)
            strict_antitone_on_apply(domain, original, left, right)
            original(left) = replacement(left)
            original(right) = replacement(right)
            original(right) < original(left)
            replacement(right) < replacement(left)
        }
    }
}

/// Monotonicity on a set is reflected by pointwise equality on that set.
theorem monotone_on_reflect_function_eq_on[Domain: PartialOrder, Value: PartialOrder](
    domain: Set[Domain], original: Domain -> Value, replacement: Domain -> Value
) {
    function_eq_on(domain, original, replacement) and is_monotone_on(domain, replacement)
    implies is_monotone_on(domain, original)
} by {
    function_eq_on_symm(domain, original, replacement)
    function_eq_on(domain, replacement, original)
    monotone_on_of_function_eq_on(domain, replacement, original)
    is_monotone_on(domain, original)
}

/// Antitonicity on a set is reflected by pointwise equality on that set.
theorem antitone_on_reflect_function_eq_on[Domain: PartialOrder, Value: PartialOrder](
    domain: Set[Domain], original: Domain -> Value, replacement: Domain -> Value
) {
    function_eq_on(domain, original, replacement) and is_antitone_on(domain, replacement)
    implies is_antitone_on(domain, original)
} by {
    function_eq_on_symm(domain, original, replacement)
    function_eq_on(domain, replacement, original)
    antitone_on_of_function_eq_on(domain, replacement, original)
    is_antitone_on(domain, original)
}

/// Strict monotonicity on a set is reflected by pointwise equality on that set.
theorem strict_monotone_on_reflect_function_eq_on[Domain: PartialOrder, Value: PartialOrder](
    domain: Set[Domain], original: Domain -> Value, replacement: Domain -> Value
) {
    function_eq_on(domain, original, replacement) and is_strict_monotone_on(domain, replacement)
    implies is_strict_monotone_on(domain, original)
} by {
    function_eq_on_symm(domain, original, replacement)
    function_eq_on(domain, replacement, original)
    strict_monotone_on_of_function_eq_on(domain, replacement, original)
    is_strict_monotone_on(domain, original)
}

/// Strict antitonicity on a set is reflected by pointwise equality on that set.
theorem strict_antitone_on_reflect_function_eq_on[Domain: PartialOrder, Value: PartialOrder](
    domain: Set[Domain], original: Domain -> Value, replacement: Domain -> Value
) {
    function_eq_on(domain, original, replacement) and is_strict_antitone_on(domain, replacement)
    implies is_strict_antitone_on(domain, original)
} by {
    function_eq_on_symm(domain, original, replacement)
    function_eq_on(domain, replacement, original)
    strict_antitone_on_of_function_eq_on(domain, replacement, original)
    is_strict_antitone_on(domain, original)
}

/// A globally monotone map is monotone on every subset.
theorem monotone_on_of_monotone[Domain: PartialOrder, Value: PartialOrder](
    domain: Set[Domain], f: Domain -> Value
) {
    is_monotone(f) implies is_monotone_on(domain, f)
} by {
    if is_monotone(f) {
        forall(left: Domain, right: Domain) {
            if domain.contains(left) and domain.contains(right) and left <= right {
                monotone_apply(f, left, right)
                f(left) <= f(right)
            }
        }
        is_monotone_on(domain, f)
    }
}

/// A globally antitone map is antitone on every subset.
theorem antitone_on_of_antitone[Domain: PartialOrder, Value: PartialOrder](
    domain: Set[Domain], f: Domain -> Value
) {
    is_antitone(f) implies is_antitone_on(domain, f)
} by {
    if is_antitone(f) {
        forall(left: Domain, right: Domain) {
            if domain.contains(left) and domain.contains(right) and left <= right {
                antitone_apply(f, left, right)
                f(right) <= f(left)
            }
        }
        is_antitone_on(domain, f)
    }
}

/// A globally strictly monotone map is strictly monotone on every subset.
theorem strict_monotone_on_of_strict_monotone[Domain: PartialOrder, Value: PartialOrder](
    domain: Set[Domain], f: Domain -> Value
) {
    is_strict_monotone(f) implies is_strict_monotone_on(domain, f)
} by {
    if is_strict_monotone(f) {
        forall(left: Domain, right: Domain) {
            if domain.contains(left) and domain.contains(right) and left < right {
                strict_monotone_apply(f, left, right)
                f(left) < f(right)
            }
        }
        is_strict_monotone_on(domain, f)
    }
}

/// A globally strictly antitone map is strictly antitone on every subset.
theorem strict_antitone_on_of_strict_antitone[Domain: PartialOrder, Value: PartialOrder](
    domain: Set[Domain], f: Domain -> Value
) {
    is_strict_antitone(f) implies is_strict_antitone_on(domain, f)
} by {
    if is_strict_antitone(f) {
        forall(left: Domain, right: Domain) {
            if domain.contains(left) and domain.contains(right) and left < right {
                strict_antitone_apply(f, left, right)
                f(right) < f(left)
            }
        }
        is_strict_antitone_on(domain, f)
    }
}

/// A globally monotone map remains monotone on a set after replacement by an equal function there.
theorem monotone_on_of_monotone_and_function_eq_on[Domain: PartialOrder, Value: PartialOrder](
    domain: Set[Domain], original: Domain -> Value, replacement: Domain -> Value
) {
    is_monotone(original) and function_eq_on(domain, original, replacement)
    implies is_monotone_on(domain, replacement)
} by {
    monotone_on_of_monotone(domain, original)
    monotone_on_of_function_eq_on(domain, original, replacement)
    is_monotone_on(domain, replacement)
}

/// A globally antitone map remains antitone on a set after replacement by an equal function there.
theorem antitone_on_of_antitone_and_function_eq_on[Domain: PartialOrder, Value: PartialOrder](
    domain: Set[Domain], original: Domain -> Value, replacement: Domain -> Value
) {
    is_antitone(original) and function_eq_on(domain, original, replacement)
    implies is_antitone_on(domain, replacement)
} by {
    antitone_on_of_antitone(domain, original)
    antitone_on_of_function_eq_on(domain, original, replacement)
    is_antitone_on(domain, replacement)
}

/// A globally strictly monotone map remains strictly monotone on a set after replacement by an equal function there.
theorem strict_monotone_on_of_strict_monotone_and_function_eq_on[Domain: PartialOrder, Value: PartialOrder](
    domain: Set[Domain], original: Domain -> Value, replacement: Domain -> Value
) {
    is_strict_monotone(original) and function_eq_on(domain, original, replacement)
    implies is_strict_monotone_on(domain, replacement)
} by {
    strict_monotone_on_of_strict_monotone(domain, original)
    strict_monotone_on_of_function_eq_on(domain, original, replacement)
    is_strict_monotone_on(domain, replacement)
}

/// A globally strictly antitone map remains strictly antitone on a set after replacement by an equal function there.
theorem strict_antitone_on_of_strict_antitone_and_function_eq_on[Domain: PartialOrder, Value: PartialOrder](
    domain: Set[Domain], original: Domain -> Value, replacement: Domain -> Value
) {
    is_strict_antitone(original) and function_eq_on(domain, original, replacement)
    implies is_strict_antitone_on(domain, replacement)
} by {
    strict_antitone_on_of_strict_antitone(domain, original)
    strict_antitone_on_of_function_eq_on(domain, original, replacement)
    is_strict_antitone_on(domain, replacement)
}

/// Both pointwise-equal functions are monotone when one is monotone on the set.
theorem monotone_on_pair_of_function_eq_on[Domain: PartialOrder, Value: PartialOrder](
    domain: Set[Domain], left: Domain -> Value, right: Domain -> Value
) {
    function_eq_on(domain, left, right) and is_monotone_on(domain, left)
    implies is_monotone_on(domain, left) and is_monotone_on(domain, right)
} by {
    monotone_on_of_function_eq_on(domain, left, right)
    is_monotone_on(domain, left)
    is_monotone_on(domain, right)
}

/// Both pointwise-equal functions are antitone when one is antitone on the set.
theorem antitone_on_pair_of_function_eq_on[Domain: PartialOrder, Value: PartialOrder](
    domain: Set[Domain], left: Domain -> Value, right: Domain -> Value
) {
    function_eq_on(domain, left, right) and is_antitone_on(domain, left)
    implies is_antitone_on(domain, left) and is_antitone_on(domain, right)
} by {
    antitone_on_of_function_eq_on(domain, left, right)
    is_antitone_on(domain, left)
    is_antitone_on(domain, right)
}

/// Both pointwise-equal functions are strictly monotone when one is strictly monotone on the set.
theorem strict_monotone_on_pair_of_function_eq_on[Domain: PartialOrder, Value: PartialOrder](
    domain: Set[Domain], left: Domain -> Value, right: Domain -> Value
) {
    function_eq_on(domain, left, right) and is_strict_monotone_on(domain, left)
    implies is_strict_monotone_on(domain, left) and is_strict_monotone_on(domain, right)
} by {
    strict_monotone_on_of_function_eq_on(domain, left, right)
    is_strict_monotone_on(domain, left)
    is_strict_monotone_on(domain, right)
}

/// Both pointwise-equal functions are strictly antitone when one is strictly antitone on the set.
theorem strict_antitone_on_pair_of_function_eq_on[Domain: PartialOrder, Value: PartialOrder](
    domain: Set[Domain], left: Domain -> Value, right: Domain -> Value
) {
    function_eq_on(domain, left, right) and is_strict_antitone_on(domain, left)
    implies is_strict_antitone_on(domain, left) and is_strict_antitone_on(domain, right)
} by {
    strict_antitone_on_of_function_eq_on(domain, left, right)
    is_strict_antitone_on(domain, left)
    is_strict_antitone_on(domain, right)
}

/// The identity map is monotone on every subset of a partial order.
theorem identity_monotone_on[Domain: PartialOrder](domain: Set[Domain]) {
    is_monotone_on(domain, identity_fn[Domain])
} by {
    identity_is_monotone[Domain]
    monotone_on_of_monotone(domain, identity_fn[Domain])
}

/// The identity map is strictly monotone on every subset of a partial order.
theorem identity_strict_monotone_on[Domain: PartialOrder](domain: Set[Domain]) {
    is_strict_monotone_on(domain, identity_fn[Domain])
} by {
    identity_is_strict_monotone[Domain]
    strict_monotone_on_of_strict_monotone(domain, identity_fn[Domain])
}

/// A constant-valued map is monotone on every subset.
theorem constant_monotone_on[Domain: PartialOrder, Value: PartialOrder](
    domain: Set[Domain], value: Value
) {
    is_monotone_on(domain, constant[Domain, Value](value))
} by {
    forall(left: Domain, right: Domain) {
        if domain.contains(left) and domain.contains(right) and left <= right {
            constant[Domain, Value](value, left) = value
            constant[Domain, Value](value, right) = value
            lte_refl(value)
            constant[Domain, Value](value, left) <= constant[Domain, Value](value, right)
        }
    }
    is_monotone_on(domain, constant[Domain, Value](value))
}

/// A constant-valued map is antitone on every subset.
theorem constant_antitone_on[Domain: PartialOrder, Value: PartialOrder](
    domain: Set[Domain], value: Value
) {
    is_antitone_on(domain, constant[Domain, Value](value))
} by {
    forall(left: Domain, right: Domain) {
        if domain.contains(left) and domain.contains(right) and left <= right {
            constant[Domain, Value](value, left) = value
            constant[Domain, Value](value, right) = value
            lte_refl(value)
            constant[Domain, Value](value, right) <= constant[Domain, Value](value, left)
        }
    }
    is_antitone_on(domain, constant[Domain, Value](value))
}

/// Composing a monotone map after a monotone map on a set gives a monotone map on that set.
theorem monotone_compose_monotone_on[Domain: PartialOrder, Middle: PartialOrder, Value: PartialOrder](
    domain: Set[Domain], outer: Middle -> Value, inner: Domain -> Middle
) {
    is_monotone(outer) and is_monotone_on(domain, inner) implies is_monotone_on(domain, compose(outer, inner))
} by {
    if is_monotone(outer) and is_monotone_on(domain, inner) {
        forall(left: Domain, right: Domain) {
            if domain.contains(left) and domain.contains(right) and left <= right {
                monotone_on_apply(domain, inner, left, right)
                inner(left) <= inner(right)
                monotone_apply(outer, inner(left), inner(right))
                outer(inner(left)) <= outer(inner(right))
                compose(outer, inner, left) = outer(inner(left))
                compose(outer, inner, right) = outer(inner(right))
                compose(outer, inner, left) <= compose(outer, inner, right)
            }
        }
        is_monotone_on(domain, compose(outer, inner))
    }
}

/// Composing an antitone map after a monotone map on a set gives an antitone map on that set.
theorem antitone_compose_monotone_on[Domain: PartialOrder, Middle: PartialOrder, Value: PartialOrder](
    domain: Set[Domain], outer: Middle -> Value, inner: Domain -> Middle
) {
    is_antitone(outer) and is_monotone_on(domain, inner) implies is_antitone_on(domain, compose(outer, inner))
} by {
    if is_antitone(outer) and is_monotone_on(domain, inner) {
        forall(left: Domain, right: Domain) {
            if domain.contains(left) and domain.contains(right) and left <= right {
                monotone_on_apply(domain, inner, left, right)
                inner(left) <= inner(right)
                antitone_apply(outer, inner(left), inner(right))
                outer(inner(right)) <= outer(inner(left))
                compose(outer, inner, right) = outer(inner(right))
                compose(outer, inner, left) = outer(inner(left))
                compose(outer, inner, right) <= compose(outer, inner, left)
            }
        }
        is_antitone_on(domain, compose(outer, inner))
    }
}

/// Composing a monotone map after an antitone map on a set gives an antitone map on that set.
theorem monotone_compose_antitone_on[Domain: PartialOrder, Middle: PartialOrder, Value: PartialOrder](
    domain: Set[Domain], outer: Middle -> Value, inner: Domain -> Middle
) {
    is_monotone(outer) and is_antitone_on(domain, inner) implies is_antitone_on(domain, compose(outer, inner))
} by {
    if is_monotone(outer) and is_antitone_on(domain, inner) {
        forall(left: Domain, right: Domain) {
            if domain.contains(left) and domain.contains(right) and left <= right {
                antitone_on_apply(domain, inner, left, right)
                inner(right) <= inner(left)
                monotone_apply(outer, inner(right), inner(left))
                outer(inner(right)) <= outer(inner(left))
                compose(outer, inner, right) = outer(inner(right))
                compose(outer, inner, left) = outer(inner(left))
                compose(outer, inner, right) <= compose(outer, inner, left)
            }
        }
        is_antitone_on(domain, compose(outer, inner))
    }
}

/// Composing an antitone map after an antitone map on a set gives a monotone map on that set.
theorem antitone_compose_antitone_on[Domain: PartialOrder, Middle: PartialOrder, Value: PartialOrder](
    domain: Set[Domain], outer: Middle -> Value, inner: Domain -> Middle
) {
    is_antitone(outer) and is_antitone_on(domain, inner) implies is_monotone_on(domain, compose(outer, inner))
} by {
    if is_antitone(outer) and is_antitone_on(domain, inner) {
        forall(left: Domain, right: Domain) {
            if domain.contains(left) and domain.contains(right) and left <= right {
                antitone_on_apply(domain, inner, left, right)
                inner(right) <= inner(left)
                antitone_apply(outer, inner(right), inner(left))
                outer(inner(left)) <= outer(inner(right))
                compose(outer, inner, left) = outer(inner(left))
                compose(outer, inner, right) = outer(inner(right))
                compose(outer, inner, left) <= compose(outer, inner, right)
            }
        }
        is_monotone_on(domain, compose(outer, inner))
    }
}

/// Composing a strictly monotone map after a strictly monotone map on a set gives a strictly monotone map.
theorem strict_monotone_compose_strict_monotone_on[Domain: PartialOrder, Middle: PartialOrder, Value: PartialOrder](
    domain: Set[Domain], outer: Middle -> Value, inner: Domain -> Middle
) {
    is_strict_monotone(outer) and is_strict_monotone_on(domain, inner)
    implies is_strict_monotone_on(domain, compose(outer, inner))
} by {
    if is_strict_monotone(outer) and is_strict_monotone_on(domain, inner) {
        forall(left: Domain, right: Domain) {
            if domain.contains(left) and domain.contains(right) and left < right {
                strict_monotone_on_apply(domain, inner, left, right)
                inner(left) < inner(right)
                strict_monotone_apply(outer, inner(left), inner(right))
                outer(inner(left)) < outer(inner(right))
                compose(outer, inner, left) = outer(inner(left))
                compose(outer, inner, right) = outer(inner(right))
                compose(outer, inner, left) < compose(outer, inner, right)
            }
        }
        is_strict_monotone_on(domain, compose(outer, inner))
    }
}

/// Composing a strictly antitone map after a strictly monotone map on a set gives a strictly antitone map.
theorem strict_antitone_compose_strict_monotone_on[Domain: PartialOrder, Middle: PartialOrder, Value: PartialOrder](
    domain: Set[Domain], outer: Middle -> Value, inner: Domain -> Middle
) {
    is_strict_antitone(outer) and is_strict_monotone_on(domain, inner)
    implies is_strict_antitone_on(domain, compose(outer, inner))
} by {
    if is_strict_antitone(outer) and is_strict_monotone_on(domain, inner) {
        forall(left: Domain, right: Domain) {
            if domain.contains(left) and domain.contains(right) and left < right {
                strict_monotone_on_apply(domain, inner, left, right)
                inner(left) < inner(right)
                strict_antitone_apply(outer, inner(left), inner(right))
                outer(inner(right)) < outer(inner(left))
                compose(outer, inner, right) = outer(inner(right))
                compose(outer, inner, left) = outer(inner(left))
                compose(outer, inner, right) < compose(outer, inner, left)
            }
        }
        is_strict_antitone_on(domain, compose(outer, inner))
    }
}

/// Composing a strictly monotone map after a strictly antitone map on a set gives a strictly antitone map.
theorem strict_monotone_compose_strict_antitone_on[Domain: PartialOrder, Middle: PartialOrder, Value: PartialOrder](
    domain: Set[Domain], outer: Middle -> Value, inner: Domain -> Middle
) {
    is_strict_monotone(outer) and is_strict_antitone_on(domain, inner)
    implies is_strict_antitone_on(domain, compose(outer, inner))
} by {
    if is_strict_monotone(outer) and is_strict_antitone_on(domain, inner) {
        forall(left: Domain, right: Domain) {
            if domain.contains(left) and domain.contains(right) and left < right {
                strict_antitone_on_apply(domain, inner, left, right)
                inner(right) < inner(left)
                strict_monotone_apply(outer, inner(right), inner(left))
                outer(inner(right)) < outer(inner(left))
                compose(outer, inner, right) = outer(inner(right))
                compose(outer, inner, left) = outer(inner(left))
                compose(outer, inner, right) < compose(outer, inner, left)
            }
        }
        is_strict_antitone_on(domain, compose(outer, inner))
    }
}

/// Composing a strictly antitone map after a strictly antitone map on a set gives a strictly monotone map.
theorem strict_antitone_compose_strict_antitone_on[Domain: PartialOrder, Middle: PartialOrder, Value: PartialOrder](
    domain: Set[Domain], outer: Middle -> Value, inner: Domain -> Middle
) {
    is_strict_antitone(outer) and is_strict_antitone_on(domain, inner)
    implies is_strict_monotone_on(domain, compose(outer, inner))
} by {
    if is_strict_antitone(outer) and is_strict_antitone_on(domain, inner) {
        forall(left: Domain, right: Domain) {
            if domain.contains(left) and domain.contains(right) and left < right {
                strict_antitone_on_apply(domain, inner, left, right)
                inner(right) < inner(left)
                strict_antitone_apply(outer, inner(right), inner(left))
                outer(inner(left)) < outer(inner(right))
                compose(outer, inner, left) = outer(inner(left))
                compose(outer, inner, right) = outer(inner(right))
                compose(outer, inner, left) < compose(outer, inner, right)
            }
        }
        is_strict_monotone_on(domain, compose(outer, inner))
    }
}

/// Composing a monotone map on the image after a monotone map on a set gives a monotone map on that set.
theorem monotone_on_compose_monotone_on_image[Domain: PartialOrder, Middle: PartialOrder, Value: PartialOrder](
    domain: Set[Domain], outer: Middle -> Value, inner: Domain -> Middle
) {
    is_monotone_on(set_image(domain, inner), outer) and is_monotone_on(domain, inner)
    implies is_monotone_on(domain, compose(outer, inner))
} by {
    if is_monotone_on(set_image(domain, inner), outer) and is_monotone_on(domain, inner) {
        forall(left: Domain, right: Domain) {
            if domain.contains(left) and domain.contains(right) and left <= right {
                monotone_on_apply(domain, inner, left, right)
                inner(left) <= inner(right)
                maps_into_set_image(domain, inner, left)
                maps_into_set_image(domain, inner, right)
                set_image(domain, inner).contains(inner(left))
                set_image(domain, inner).contains(inner(right))
                monotone_on_apply(set_image(domain, inner), outer, inner(left), inner(right))
                outer(inner(left)) <= outer(inner(right))
                compose(outer, inner, left) = outer(inner(left))
                compose(outer, inner, right) = outer(inner(right))
                compose(outer, inner, left) <= compose(outer, inner, right)
            }
        }
        is_monotone_on(domain, compose(outer, inner))
    }
}

/// Composing an antitone map on the image after a monotone map on a set gives an antitone map on that set.
theorem antitone_on_compose_monotone_on_image[Domain: PartialOrder, Middle: PartialOrder, Value: PartialOrder](
    domain: Set[Domain], outer: Middle -> Value, inner: Domain -> Middle
) {
    is_antitone_on(set_image(domain, inner), outer) and is_monotone_on(domain, inner)
    implies is_antitone_on(domain, compose(outer, inner))
} by {
    if is_antitone_on(set_image(domain, inner), outer) and is_monotone_on(domain, inner) {
        forall(left: Domain, right: Domain) {
            if domain.contains(left) and domain.contains(right) and left <= right {
                monotone_on_apply(domain, inner, left, right)
                inner(left) <= inner(right)
                maps_into_set_image(domain, inner, left)
                maps_into_set_image(domain, inner, right)
                set_image(domain, inner).contains(inner(left))
                set_image(domain, inner).contains(inner(right))
                antitone_on_apply(set_image(domain, inner), outer, inner(left), inner(right))
                outer(inner(right)) <= outer(inner(left))
                compose(outer, inner, right) = outer(inner(right))
                compose(outer, inner, left) = outer(inner(left))
                compose(outer, inner, right) <= compose(outer, inner, left)
            }
        }
        is_antitone_on(domain, compose(outer, inner))
    }
}

/// Composing a monotone map on the image after an antitone map on a set gives an antitone map on that set.
theorem monotone_on_compose_antitone_on_image[Domain: PartialOrder, Middle: PartialOrder, Value: PartialOrder](
    domain: Set[Domain], outer: Middle -> Value, inner: Domain -> Middle
) {
    is_monotone_on(set_image(domain, inner), outer) and is_antitone_on(domain, inner)
    implies is_antitone_on(domain, compose(outer, inner))
} by {
    if is_monotone_on(set_image(domain, inner), outer) and is_antitone_on(domain, inner) {
        forall(left: Domain, right: Domain) {
            if domain.contains(left) and domain.contains(right) and left <= right {
                antitone_on_apply(domain, inner, left, right)
                inner(right) <= inner(left)
                maps_into_set_image(domain, inner, left)
                maps_into_set_image(domain, inner, right)
                set_image(domain, inner).contains(inner(left))
                set_image(domain, inner).contains(inner(right))
                monotone_on_apply(set_image(domain, inner), outer, inner(right), inner(left))
                outer(inner(right)) <= outer(inner(left))
                compose(outer, inner, right) = outer(inner(right))
                compose(outer, inner, left) = outer(inner(left))
                compose(outer, inner, right) <= compose(outer, inner, left)
            }
        }
        is_antitone_on(domain, compose(outer, inner))
    }
}

/// Composing an antitone map on the image after an antitone map on a set gives a monotone map on that set.
theorem antitone_on_compose_antitone_on_image[Domain: PartialOrder, Middle: PartialOrder, Value: PartialOrder](
    domain: Set[Domain], outer: Middle -> Value, inner: Domain -> Middle
) {
    is_antitone_on(set_image(domain, inner), outer) and is_antitone_on(domain, inner)
    implies is_monotone_on(domain, compose(outer, inner))
} by {
    if is_antitone_on(set_image(domain, inner), outer) and is_antitone_on(domain, inner) {
        forall(left: Domain, right: Domain) {
            if domain.contains(left) and domain.contains(right) and left <= right {
                antitone_on_apply(domain, inner, left, right)
                inner(right) <= inner(left)
                maps_into_set_image(domain, inner, left)
                maps_into_set_image(domain, inner, right)
                set_image(domain, inner).contains(inner(left))
                set_image(domain, inner).contains(inner(right))
                antitone_on_apply(set_image(domain, inner), outer, inner(right), inner(left))
                outer(inner(left)) <= outer(inner(right))
                compose(outer, inner, left) = outer(inner(left))
                compose(outer, inner, right) = outer(inner(right))
                compose(outer, inner, left) <= compose(outer, inner, right)
            }
        }
        is_monotone_on(domain, compose(outer, inner))
    }
}

/// Composing a strictly monotone map on the image after a strictly monotone map gives a strictly monotone map.
theorem strict_monotone_on_compose_strict_monotone_on_image[Domain: PartialOrder, Middle: PartialOrder, Value: PartialOrder](
    domain: Set[Domain], outer: Middle -> Value, inner: Domain -> Middle
) {
    is_strict_monotone_on(set_image(domain, inner), outer) and is_strict_monotone_on(domain, inner)
    implies is_strict_monotone_on(domain, compose(outer, inner))
} by {
    if is_strict_monotone_on(set_image(domain, inner), outer) and is_strict_monotone_on(domain, inner) {
        forall(left: Domain, right: Domain) {
            if domain.contains(left) and domain.contains(right) and left < right {
                strict_monotone_on_apply(domain, inner, left, right)
                inner(left) < inner(right)
                maps_into_set_image(domain, inner, left)
                maps_into_set_image(domain, inner, right)
                set_image(domain, inner).contains(inner(left))
                set_image(domain, inner).contains(inner(right))
                strict_monotone_on_apply(set_image(domain, inner), outer, inner(left), inner(right))
                outer(inner(left)) < outer(inner(right))
                compose(outer, inner, left) = outer(inner(left))
                compose(outer, inner, right) = outer(inner(right))
                compose(outer, inner, left) < compose(outer, inner, right)
            }
        }
        is_strict_monotone_on(domain, compose(outer, inner))
    }
}

/// Composing a strictly antitone map on the image after a strictly monotone map gives a strictly antitone map.
theorem strict_antitone_on_compose_strict_monotone_on_image[Domain: PartialOrder, Middle: PartialOrder, Value: PartialOrder](
    domain: Set[Domain], outer: Middle -> Value, inner: Domain -> Middle
) {
    is_strict_antitone_on(set_image(domain, inner), outer) and is_strict_monotone_on(domain, inner)
    implies is_strict_antitone_on(domain, compose(outer, inner))
} by {
    if is_strict_antitone_on(set_image(domain, inner), outer) and is_strict_monotone_on(domain, inner) {
        forall(left: Domain, right: Domain) {
            if domain.contains(left) and domain.contains(right) and left < right {
                strict_monotone_on_apply(domain, inner, left, right)
                inner(left) < inner(right)
                maps_into_set_image(domain, inner, left)
                maps_into_set_image(domain, inner, right)
                set_image(domain, inner).contains(inner(left))
                set_image(domain, inner).contains(inner(right))
                strict_antitone_on_apply(set_image(domain, inner), outer, inner(left), inner(right))
                outer(inner(right)) < outer(inner(left))
                compose(outer, inner, right) = outer(inner(right))
                compose(outer, inner, left) = outer(inner(left))
                compose(outer, inner, right) < compose(outer, inner, left)
            }
        }
        is_strict_antitone_on(domain, compose(outer, inner))
    }
}

/// Composing a strictly monotone map on the image after a strictly antitone map gives a strictly antitone map.
theorem strict_monotone_on_compose_strict_antitone_on_image[Domain: PartialOrder, Middle: PartialOrder, Value: PartialOrder](
    domain: Set[Domain], outer: Middle -> Value, inner: Domain -> Middle
) {
    is_strict_monotone_on(set_image(domain, inner), outer) and is_strict_antitone_on(domain, inner)
    implies is_strict_antitone_on(domain, compose(outer, inner))
} by {
    if is_strict_monotone_on(set_image(domain, inner), outer) and is_strict_antitone_on(domain, inner) {
        forall(left: Domain, right: Domain) {
            if domain.contains(left) and domain.contains(right) and left < right {
                strict_antitone_on_apply(domain, inner, left, right)
                inner(right) < inner(left)
                maps_into_set_image(domain, inner, left)
                maps_into_set_image(domain, inner, right)
                set_image(domain, inner).contains(inner(left))
                set_image(domain, inner).contains(inner(right))
                strict_monotone_on_apply(set_image(domain, inner), outer, inner(right), inner(left))
                outer(inner(right)) < outer(inner(left))
                compose(outer, inner, right) = outer(inner(right))
                compose(outer, inner, left) = outer(inner(left))
                compose(outer, inner, right) < compose(outer, inner, left)
            }
        }
        is_strict_antitone_on(domain, compose(outer, inner))
    }
}

/// Composing a strictly antitone map on the image after a strictly antitone map gives a strictly monotone map.
theorem strict_antitone_on_compose_strict_antitone_on_image[Domain: PartialOrder, Middle: PartialOrder, Value: PartialOrder](
    domain: Set[Domain], outer: Middle -> Value, inner: Domain -> Middle
) {
    is_strict_antitone_on(set_image(domain, inner), outer) and is_strict_antitone_on(domain, inner)
    implies is_strict_monotone_on(domain, compose(outer, inner))
} by {
    if is_strict_antitone_on(set_image(domain, inner), outer) and is_strict_antitone_on(domain, inner) {
        forall(left: Domain, right: Domain) {
            if domain.contains(left) and domain.contains(right) and left < right {
                strict_antitone_on_apply(domain, inner, left, right)
                inner(right) < inner(left)
                maps_into_set_image(domain, inner, left)
                maps_into_set_image(domain, inner, right)
                set_image(domain, inner).contains(inner(left))
                set_image(domain, inner).contains(inner(right))
                strict_antitone_on_apply(set_image(domain, inner), outer, inner(right), inner(left))
                outer(inner(left)) < outer(inner(right))
                compose(outer, inner, left) = outer(inner(left))
                compose(outer, inner, right) = outer(inner(right))
                compose(outer, inner, left) < compose(outer, inner, right)
            }
        }
        is_strict_monotone_on(domain, compose(outer, inner))
    }
}

/// Strict monotonicity on a set implies non-strict monotonicity on that set.
theorem strict_monotone_on_imp_monotone_on[Domain: PartialOrder, Value: PartialOrder](
    domain: Set[Domain], f: Domain -> Value
) {
    is_strict_monotone_on(domain, f) implies is_monotone_on(domain, f)
} by {
    if is_strict_monotone_on(domain, f) {
        forall(left: Domain, right: Domain) {
            if domain.contains(left) and domain.contains(right) and left <= right {
                if left = right {
                    lte_refl(f(right))
                    f(left) <= f(right)
                }
                if left != right {
                    left < right
                    strict_monotone_on_apply(domain, f, left, right)
                    f(left) < f(right)
                    lt_imp_lte(f(left), f(right))
                    f(left) <= f(right)
                }
            }
        }
        is_monotone_on(domain, f)
    }
}

/// Strict antitonicity on a set implies non-strict antitonicity on that set.
theorem strict_antitone_on_imp_antitone_on[Domain: PartialOrder, Value: PartialOrder](
    domain: Set[Domain], f: Domain -> Value
) {
    is_strict_antitone_on(domain, f) implies is_antitone_on(domain, f)
} by {
    if is_strict_antitone_on(domain, f) {
        forall(left: Domain, right: Domain) {
            if domain.contains(left) and domain.contains(right) and left <= right {
                if left = right {
                    lte_refl(f(left))
                    f(right) <= f(left)
                }
                if left != right {
                    left < right
                    strict_antitone_on_apply(domain, f, left, right)
                    f(right) < f(left)
                    lt_imp_lte(f(right), f(left))
                    f(right) <= f(left)
                }
            }
        }
        is_antitone_on(domain, f)
    }
}

/// A function equal on the set to a strictly monotone function is monotone there.
theorem monotone_on_of_strict_monotone_on_and_function_eq_on[Domain: PartialOrder, Value: PartialOrder](
    domain: Set[Domain], original: Domain -> Value, replacement: Domain -> Value
) {
    is_strict_monotone_on(domain, original) and function_eq_on(domain, original, replacement)
    implies is_monotone_on(domain, replacement)
} by {
    strict_monotone_on_of_function_eq_on(domain, original, replacement)
    is_strict_monotone_on(domain, replacement)
    strict_monotone_on_imp_monotone_on(domain, replacement)
    is_monotone_on(domain, replacement)
}

/// A function equal on the set to a strictly antitone function is antitone there.
theorem antitone_on_of_strict_antitone_on_and_function_eq_on[Domain: PartialOrder, Value: PartialOrder](
    domain: Set[Domain], original: Domain -> Value, replacement: Domain -> Value
) {
    is_strict_antitone_on(domain, original) and function_eq_on(domain, original, replacement)
    implies is_antitone_on(domain, replacement)
} by {
    strict_antitone_on_of_function_eq_on(domain, original, replacement)
    is_strict_antitone_on(domain, replacement)
    strict_antitone_on_imp_antitone_on(domain, replacement)
    is_antitone_on(domain, replacement)
}

/// A function equal on the set to a globally strictly monotone function is monotone there.
theorem monotone_on_of_strict_monotone_and_function_eq_on[Domain: PartialOrder, Value: PartialOrder](
    domain: Set[Domain], original: Domain -> Value, replacement: Domain -> Value
) {
    is_strict_monotone(original) and function_eq_on(domain, original, replacement)
    implies is_monotone_on(domain, replacement)
} by {
    strict_monotone_on_of_strict_monotone_and_function_eq_on(domain, original, replacement)
    is_strict_monotone_on(domain, replacement)
    strict_monotone_on_imp_monotone_on(domain, replacement)
    is_monotone_on(domain, replacement)
}

/// A function equal on the set to a globally strictly antitone function is antitone there.
theorem antitone_on_of_strict_antitone_and_function_eq_on[Domain: PartialOrder, Value: PartialOrder](
    domain: Set[Domain], original: Domain -> Value, replacement: Domain -> Value
) {
    is_strict_antitone(original) and function_eq_on(domain, original, replacement)
    implies is_antitone_on(domain, replacement)
} by {
    strict_antitone_on_of_strict_antitone_and_function_eq_on(domain, original, replacement)
    is_strict_antitone_on(domain, replacement)
    strict_antitone_on_imp_antitone_on(domain, replacement)
    is_antitone_on(domain, replacement)
}

/// Monotonicity on a set is preserved by restricting the domain.
theorem monotone_on_subset[Domain: PartialOrder, Value: PartialOrder](
    subset: Set[Domain], superset: Set[Domain], f: Domain -> Value
) {
    subset.subset(superset) and is_monotone_on(superset, f) implies is_monotone_on(subset, f)
} by {
    if subset.subset(superset) and is_monotone_on(superset, f) {
        forall(left: Domain, right: Domain) {
            if subset.contains(left) and subset.contains(right) and left <= right {
                superset.contains(left)
                superset.contains(right)
                monotone_on_apply(superset, f, left, right)
                f(left) <= f(right)
            }
        }
        is_monotone_on(subset, f)
    }
}

/// Antitonicity on a set is preserved by restricting the domain.
theorem antitone_on_subset[Domain: PartialOrder, Value: PartialOrder](
    subset: Set[Domain], superset: Set[Domain], f: Domain -> Value
) {
    subset.subset(superset) and is_antitone_on(superset, f) implies is_antitone_on(subset, f)
} by {
    if subset.subset(superset) and is_antitone_on(superset, f) {
        forall(left: Domain, right: Domain) {
            if subset.contains(left) and subset.contains(right) and left <= right {
                superset.contains(left)
                superset.contains(right)
                antitone_on_apply(superset, f, left, right)
                f(right) <= f(left)
            }
        }
        is_antitone_on(subset, f)
    }
}

/// Strict monotonicity on a set is preserved by restricting the domain.
theorem strict_monotone_on_subset[Domain: PartialOrder, Value: PartialOrder](
    subset: Set[Domain], superset: Set[Domain], f: Domain -> Value
) {
    subset.subset(superset) and is_strict_monotone_on(superset, f) implies is_strict_monotone_on(subset, f)
} by {
    if subset.subset(superset) and is_strict_monotone_on(superset, f) {
        forall(left: Domain, right: Domain) {
            if subset.contains(left) and subset.contains(right) and left < right {
                superset.contains(left)
                superset.contains(right)
                strict_monotone_on_apply(superset, f, left, right)
                f(left) < f(right)
            }
        }
        is_strict_monotone_on(subset, f)
    }
}

/// Strict antitonicity on a set is preserved by restricting the domain.
theorem strict_antitone_on_subset[Domain: PartialOrder, Value: PartialOrder](
    subset: Set[Domain], superset: Set[Domain], f: Domain -> Value
) {
    subset.subset(superset) and is_strict_antitone_on(superset, f) implies is_strict_antitone_on(subset, f)
} by {
    if subset.subset(superset) and is_strict_antitone_on(superset, f) {
        forall(left: Domain, right: Domain) {
            if subset.contains(left) and subset.contains(right) and left < right {
                superset.contains(left)
                superset.contains(right)
                strict_antitone_on_apply(superset, f, left, right)
                f(right) < f(left)
            }
        }
        is_strict_antitone_on(subset, f)
    }
}
