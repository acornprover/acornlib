/// Bounds and extrema for functions with partially ordered values.

from order import PartialOrder
from data.basic.set_base import Set

/// The value `bound` is an upper bound for `f` on `domain`.
define is_upper_bound_on[Domain, Value: PartialOrder](
    domain: Set[Domain], f: Domain -> Value, bound: Value
) -> Bool {
    forall(point: Domain) {
        domain.contains(point) implies f(point) <= bound
    }
}

/// The value `bound` is a lower bound for `f` on `domain`.
define is_lower_bound_on[Domain, Value: PartialOrder](
    domain: Set[Domain], f: Domain -> Value, bound: Value
) -> Bool {
    forall(point: Domain) {
        domain.contains(point) implies bound <= f(point)
    }
}

/// The function `f` has an upper bound on `domain`.
define is_bounded_above_on[Domain, Value: PartialOrder](
    domain: Set[Domain], f: Domain -> Value
) -> Bool {
    exists(bound: Value) {
        is_upper_bound_on(domain, f, bound)
    }
}

/// The function `f` has a lower bound on `domain`.
define is_bounded_below_on[Domain, Value: PartialOrder](
    domain: Set[Domain], f: Domain -> Value
) -> Bool {
    exists(bound: Value) {
        is_lower_bound_on(domain, f, bound)
    }
}

/// The function `f` has both an upper and a lower bound on `domain`.
define is_bounded_on[Domain, Value: PartialOrder](
    domain: Set[Domain], f: Domain -> Value
) -> Bool {
    is_bounded_above_on(domain, f) and is_bounded_below_on(domain, f)
}

/// An upper bound dominates the value at every point of its domain.
theorem upper_bound_on_apply[Domain, Value: PartialOrder](
    domain: Set[Domain], f: Domain -> Value, bound: Value, point: Domain
) {
    is_upper_bound_on(domain, f, bound) and domain.contains(point) implies f(point) <= bound
} by {
    if is_upper_bound_on(domain, f, bound) and domain.contains(point) {
        is_upper_bound_on(domain, f, bound) = forall(candidate: Domain) {
            domain.contains(candidate) implies f(candidate) <= bound
        }
        domain.contains(point) implies f(point) <= bound
        f(point) <= bound
    }
}

/// A lower bound is dominated by the value at every point of its domain.
theorem lower_bound_on_apply[Domain, Value: PartialOrder](
    domain: Set[Domain], f: Domain -> Value, bound: Value, point: Domain
) {
    is_lower_bound_on(domain, f, bound) and domain.contains(point) implies bound <= f(point)
} by {
    if is_lower_bound_on(domain, f, bound) and domain.contains(point) {
        is_lower_bound_on(domain, f, bound) = forall(candidate: Domain) {
            domain.contains(candidate) implies bound <= f(candidate)
        }
        domain.contains(point) implies bound <= f(point)
        bound <= f(point)
    }
}

/// The function `f` attains a maximum at `point` on `domain`.
define attains_maximum_at[Domain, Value: PartialOrder](
    domain: Set[Domain], f: Domain -> Value, point: Domain
) -> Bool {
    domain.contains(point) and is_upper_bound_on(domain, f, f(point))
}

/// The function `f` attains a minimum at `point` on `domain`.
define attains_minimum_at[Domain, Value: PartialOrder](
    domain: Set[Domain], f: Domain -> Value, point: Domain
) -> Bool {
    domain.contains(point) and is_lower_bound_on(domain, f, f(point))
}

/// The function `f` attains a maximum somewhere on `domain`.
define attains_maximum_on[Domain, Value: PartialOrder](
    domain: Set[Domain], f: Domain -> Value
) -> Bool {
    exists(point: Domain) {
        attains_maximum_at(domain, f, point)
    }
}

/// The function `f` attains a minimum somewhere on `domain`.
define attains_minimum_on[Domain, Value: PartialOrder](
    domain: Set[Domain], f: Domain -> Value
) -> Bool {
    exists(point: Domain) {
        attains_minimum_at(domain, f, point)
    }
}

/// A maximum value is an upper bound.
theorem maximum_value_is_upper_bound_on[Domain, Value: PartialOrder](
    domain: Set[Domain], f: Domain -> Value, point: Domain
) {
    attains_maximum_at(domain, f, point) implies is_upper_bound_on(domain, f, f(point))
} by {
    if attains_maximum_at(domain, f, point) {
        is_upper_bound_on(domain, f, f(point))
    }
}

/// A minimum value is a lower bound.
theorem minimum_value_is_lower_bound_on[Domain, Value: PartialOrder](
    domain: Set[Domain], f: Domain -> Value, point: Domain
) {
    attains_minimum_at(domain, f, point) implies is_lower_bound_on(domain, f, f(point))
} by {
    if attains_minimum_at(domain, f, point) {
        is_lower_bound_on(domain, f, f(point))
    }
}

/// A function which attains a maximum is bounded above.
theorem attains_maximum_imp_bounded_above[Domain, Value: PartialOrder](
    domain: Set[Domain], f: Domain -> Value
) {
    attains_maximum_on(domain, f) implies is_bounded_above_on(domain, f)
} by {
    if attains_maximum_on(domain, f) {
        let point: Domain satisfy {
            attains_maximum_at(domain, f, point)
        }
        maximum_value_is_upper_bound_on(domain, f, point)
        is_upper_bound_on(domain, f, f(point))
        exists(bound: Value) {
            is_upper_bound_on(domain, f, bound)
        }
        is_bounded_above_on(domain, f)
    }
}

/// A function which attains a minimum is bounded below.
theorem attains_minimum_imp_bounded_below[Domain, Value: PartialOrder](
    domain: Set[Domain], f: Domain -> Value
) {
    attains_minimum_on(domain, f) implies is_bounded_below_on(domain, f)
} by {
    if attains_minimum_on(domain, f) {
        let point: Domain satisfy {
            attains_minimum_at(domain, f, point)
        }
        minimum_value_is_lower_bound_on(domain, f, point)
        is_lower_bound_on(domain, f, f(point))
        exists(bound: Value) {
            is_lower_bound_on(domain, f, bound)
        }
        is_bounded_below_on(domain, f)
    }
}

/// An upper bound on a set remains an upper bound after restricting the domain.
theorem upper_bound_on_subset[Domain, Value: PartialOrder](
    subset: Set[Domain], superset: Set[Domain], f: Domain -> Value, bound: Value
) {
    subset.subset(superset) and is_upper_bound_on(superset, f, bound)
    implies is_upper_bound_on(subset, f, bound)
} by {
    if subset.subset(superset) and is_upper_bound_on(superset, f, bound) {
        forall(point: Domain) {
            if subset.contains(point) {
                subset.subset(superset) = forall(candidate: Domain) {
                    subset.contains(candidate) implies superset.contains(candidate)
                }
                subset.contains(point) implies superset.contains(point)
                superset.contains(point)
                upper_bound_on_apply(superset, f, bound, point)
                f(point) <= bound
            }
        }
        is_upper_bound_on(subset, f, bound)
    }
}

/// A lower bound on a set remains a lower bound after restricting the domain.
theorem lower_bound_on_subset[Domain, Value: PartialOrder](
    subset: Set[Domain], superset: Set[Domain], f: Domain -> Value, bound: Value
) {
    subset.subset(superset) and is_lower_bound_on(superset, f, bound)
    implies is_lower_bound_on(subset, f, bound)
} by {
    if subset.subset(superset) and is_lower_bound_on(superset, f, bound) {
        forall(point: Domain) {
            if subset.contains(point) {
                subset.subset(superset) = forall(candidate: Domain) {
                    subset.contains(candidate) implies superset.contains(candidate)
                }
                subset.contains(point) implies superset.contains(point)
                superset.contains(point)
                lower_bound_on_apply(superset, f, bound, point)
                bound <= f(point)
            }
        }
        is_lower_bound_on(subset, f, bound)
    }
}
