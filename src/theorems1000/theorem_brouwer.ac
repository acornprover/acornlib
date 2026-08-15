from real import Real, continuous, identity_fn, continuous_pointwise_add,
    continuous_pointwise_neg, identity_function_is_continuous,
    intermediate_value_closed_interval, sub_zero_imp_eq
from order_set import closed_interval_set, closed_interval_set_contains_lower,
    closed_interval_set_contains_upper, closed_interval_set_lower_le, closed_interval_set_le_upper
from algebra.add_ordered_group import add_le_add_right, neg_le_neg
from order import lt_imp_lte
from data.basic.function_algebra import pointwise_add, pointwise_neg, pointwise_add_apply,
    pointwise_neg_apply

// Brouwer's fixed-point theorem (1911): every continuous map from the
// closed unit ball of R^n to itself has a fixed point.  For n = 2, the
// statement is: every continuous map
//     f: {(x, y) : x^2 + y^2 <= 1} -> {(x, y) : x^2 + y^2 <= 1}
// has a point p with f(p) = p.  The theorem was proved by L. E. J. Brouwer
// in 1911 for n = 3 and for general n in 1912; the one-dimensional case is
// the intermediate value theorem.  It has far-reaching consequences:
// the fundamental theorem of algebra, the Perron-Frobenius theorem for
// positive matrices, the existence of Nash equilibria in game theory, and
// the Schauder fixed-point theorem in infinite dimensions all follow from
// it (or from its infinite-dimensional relatives).
//
// The library has the closed unit disk formalized as the predicate
// `in_closed_disk(Complex.0, Real.1, z)` (in `complex/complex_region.ac`),
// but it does not yet define continuity of functions on the complex plane,
// so the statement below uses an informal `continuous(f)` that would need a
// formal definition (epsilon-delta continuity on the disk).  The theorem is
// recorded here but not proved.
//
// Proof sketch: the standard proofs use either the no-retraction theorem
// (there is no continuous map from the disk to its boundary circle that
// fixes the boundary) plus homology or the fundamental group of the circle,
// or an analytic argument with the Brouwer degree of a map (the degree of
// f - id at 0 is nonzero, so 0 is in the image of f - id).  All of these
// need the topology of the plane (the fundamental group pi_1(S^1) = Z, or
// invariance of domain), which is not in the library.
//
// theorem theorems1000_brouwer_disk(f: Complex -> Complex) {
//     (forall(z: Complex) {
//         in_closed_disk(Complex.0, Real.1, z) implies
//             in_closed_disk(Complex.0, Real.1, f(z))
//     } and continuous(f)) implies
//         exists(z: Complex) {
//             in_closed_disk(Complex.0, Real.1, z) and f(z) = z
//         }
// }

// The one-dimensional case of Brouwer's fixed point theorem — every
// continuous self-map of the closed unit interval has a fixed point — is
// proved below as `theorems1000_brouwer_interval`.  It is the classical
// corollary of the intermediate value theorem applied to
// `g(x) = x - f(x)`: the map `g` is nonpositive at 0 and nonnegative at 1
// (because `f` maps the interval into itself), so it vanishes at some
// `root` in `[0, 1]`, and `g(root) = 0` reads `f(root) = root`.  The
// library proves the same statement as `unit_interval_fixed_point` in
// `real/fixed_point_theorems.ac` (the one-dimensional case of Brouwer's
// theorem, for the unit interval); the proof below develops it from the
// public real analysis interface.

/// Brouwer's fixed point theorem, one-dimensional case: a continuous map
/// of the closed unit interval into itself has a fixed point.
theorem theorems1000_brouwer_interval(f: Real -> Real) {
    continuous(f) and
    (forall(x: Real) {
        closed_interval_set(Real.0, Real.1).contains(x) implies
            closed_interval_set(Real.0, Real.1).contains(f(x))
    })
    implies exists(point: Real) {
        closed_interval_set(Real.0, Real.1).contains(point) and f(point) = point
    }
} by {
    if continuous(f) and (forall(x: Real) {
        closed_interval_set(Real.0, Real.1).contains(x) implies
            closed_interval_set(Real.0, Real.1).contains(f(x))
    }) {
        // g = id + (-f) is continuous
        continuous_pointwise_neg(f)
        continuous(pointwise_neg(f))
        identity_function_is_continuous
        continuous(identity_fn[Real])
        continuous_pointwise_add(identity_fn[Real], pointwise_neg(f))
        continuous(pointwise_add(identity_fn[Real], pointwise_neg(f)))
        // g(0) = 0 - f(0) <= 0: from 0 <= f(0)
        Real.0 < Real.1
        lt_imp_lte[Real](Real.0, Real.1)
        Real.0 <= Real.1
        closed_interval_set_contains_lower[Real](Real.0, Real.1)
        closed_interval_set(Real.0, Real.1).contains(Real.0)
        closed_interval_set(Real.0, Real.1).contains(f(Real.0))
        closed_interval_set_lower_le[Real](Real.0, Real.1, f(Real.0))
        Real.0 <= f(Real.0)
        neg_le_neg[Real](f(Real.0), Real.0)
        -f(Real.0) <= Real.0
        add_le_add_right[Real](Real.0, -f(Real.0), Real.0)
        Real.0 + -f(Real.0) <= Real.0 + Real.0
        Real.0 + -f(Real.0) <= Real.0
        pointwise_neg_apply[Real, Real](f, Real.0)
        pointwise_neg(f, Real.0) = -f(Real.0)
        pointwise_add_apply[Real, Real](identity_fn[Real], pointwise_neg(f), Real.0)
        pointwise_add(identity_fn[Real], pointwise_neg(f), Real.0) =
            identity_fn[Real](Real.0) + pointwise_neg(f, Real.0)
        identity_fn[Real](Real.0) = Real.0
        pointwise_add(identity_fn[Real], pointwise_neg(f), Real.0) = Real.0 + -f(Real.0)
        pointwise_add(identity_fn[Real], pointwise_neg(f), Real.0) <= Real.0
        // g(1) = 1 - f(1) >= 0: from f(1) <= 1
        closed_interval_set_contains_upper[Real](Real.0, Real.1)
        closed_interval_set(Real.0, Real.1).contains(Real.1)
        closed_interval_set(Real.0, Real.1).contains(f(Real.1))
        closed_interval_set_le_upper[Real](Real.0, Real.1, f(Real.1))
        f(Real.1) <= Real.1
        add_le_add_right[Real](f(Real.1), Real.1, -f(Real.1))
        f(Real.1) + -f(Real.1) <= Real.1 + -f(Real.1)
        f(Real.1) + -f(Real.1) = Real.0
        Real.1 + -f(Real.1) = Real.1 - f(Real.1)
        Real.0 <= Real.1 - f(Real.1)
        pointwise_neg_apply[Real, Real](f, Real.1)
        pointwise_neg(f, Real.1) = -f(Real.1)
        pointwise_add_apply[Real, Real](identity_fn[Real], pointwise_neg(f), Real.1)
        pointwise_add(identity_fn[Real], pointwise_neg(f), Real.1) =
            identity_fn[Real](Real.1) + pointwise_neg(f, Real.1)
        identity_fn[Real](Real.1) = Real.1
        pointwise_add(identity_fn[Real], pointwise_neg(f), Real.1) = Real.1 - f(Real.1)
        Real.0 <= pointwise_add(identity_fn[Real], pointwise_neg(f), Real.1)
        // the intermediate value theorem gives a root of g in [0, 1]
        intermediate_value_closed_interval(pointwise_add(identity_fn[Real], pointwise_neg(f)),
            Real.0, Real.1, Real.0)
        exists(root: Real) {
            closed_interval_set(Real.0, Real.1).contains(root) and
                pointwise_add(identity_fn[Real], pointwise_neg(f), root) = Real.0
        }
        let root: Real satisfy {
            closed_interval_set(Real.0, Real.1).contains(root) and
                pointwise_add(identity_fn[Real], pointwise_neg(f), root) = Real.0
        }
        closed_interval_set(Real.0, Real.1).contains(root)
        pointwise_add(identity_fn[Real], pointwise_neg(f), root) = Real.0
        pointwise_neg_apply[Real, Real](f, root)
        pointwise_neg(f, root) = -f(root)
        pointwise_add_apply[Real, Real](identity_fn[Real], pointwise_neg(f), root)
        pointwise_add(identity_fn[Real], pointwise_neg(f), root) =
            identity_fn[Real](root) + pointwise_neg(f, root)
        identity_fn[Real](root) = root
        pointwise_add(identity_fn[Real], pointwise_neg(f), root) = root - f(root)
        root - f(root) = Real.0
        sub_zero_imp_eq(root, f(root))
        root = f(root)
        closed_interval_set(Real.0, Real.1).contains(root) and f(root) = root
        exists(fixed: Real) {
            closed_interval_set(Real.0, Real.1).contains(fixed) and f(fixed) = fixed
        }
    }
}
