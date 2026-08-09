from pair import Pair
from real import Real
from top100 import r2_sub
from theorems1000.r2_helpers import r2_smul, r2_cross

// Monge's theorem (also called d'Alembert's theorem): given three circles
// in the plane, the three external homothety centers, taken pairwise, are
// collinear.
//
// The circles have centers `o1`, `o2`, `o3` and radii `r1`, `r2`, `r3`.  The
// external homothety center `h12` of the circles `(o1, r1)` and `(o2, r2)`
// is the point on the line of centers such that the homothety centered at
// `h12` with ratio `r2 / r1` maps the first circle to the second:
//     o2 - h12 = (r2 / r1) (o1 - h12),
// equivalently, clearing the ratio,
//     (r2 - r1) h12 = r2 o1 - r1 o2,   i.e.   r2 (h12 - o1) = r1 (h12 - o2).
// The three external centers are collinear, which is the vanishing of the
// cross product
//     cross(h12 - h23, h31 - h23) = 0.
// The statement below records the three defining equations of the external
// centers as hypotheses, together with the non-degeneracy conditions
// `r2 - r1 != 0`, `r3 - r2 != 0`, `r1 - r3 != 0` (no two circles have equal
// radii; the equal-radius case sends the external center to infinity), and
// concludes the collinearity.
//
// Proof sketch: introduce the vector
//     D = o1 (r3 - r2) + o2 (r1 - r3) + o3 (r2 - r1).
// A direct computation from the three defining equations gives
//     h12 - h23 = r2 D / ((r2 - r1) (r3 - r2)),
//     h23 - h31 = r3 D / ((r3 - r2) (r1 - r3)),
//     h31 - h12 = r1 D / ((r1 - r3) (r2 - r1)),
// so the three pairwise difference vectors are all scalar multiples of the
// single vector `D`, hence parallel; consequently
//     cross(h12 - h23, h31 - h23) =
//         cross(r2 D / d1, -r3 D / d2) = -r2 r3 / (d1 d2) cross(D, D) = 0.
// Formally one solves each defining equation for `h12`, `h23`, `h31` using
// the real field inverse (each `r_i - r_j` is nonzero by hypothesis), clears
// the denominators, and applies the cross-product bilinearity lemmas
// (`r2_cross_add_left`, `r2_cross_smul_left`, `r2_cross_self_zero`); the
// proof is a long but purely algebraic chain with division, which is why the
// statement is recorded here with the computation left as a sketch.
//
// theorem theorems1000_monge(
//     o1: Pair[Real, Real], o2: Pair[Real, Real], o3: Pair[Real, Real],
//     r1: Real, r2: Real, r3: Real,
//     h12: Pair[Real, Real], h23: Pair[Real, Real], h31: Pair[Real, Real]
// ) {
//     r2_smul(r2 - r1, h12) = r2_sub(r2_smul(r2, o1), r2_smul(r1, o2)) and
//     r2_smul(r3 - r2, h23) = r2_sub(r2_smul(r3, o2), r2_smul(r2, o3)) and
//     r2_smul(r1 - r3, h31) = r2_sub(r2_smul(r1, o3), r2_smul(r3, o1)) and
//     r2 - r1 != Real.0 and r3 - r2 != Real.0 and r1 - r3 != Real.0
//     implies
//     r2_cross(r2_sub(h12, h23), r2_sub(h31, h23)) = Real.0
// }
