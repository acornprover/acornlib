from pair import Pair, pair_ext
from real import Real
from top100 import r2_add, r2_sub, r2_norm_sq
from theorems1000.r2_helpers import r2_cross, r2_double_pair_rearrange, r2_double_eq_double,
    real_add_pair_rearrange

// Newton's theorem (quadrilateral): in a tangential quadrilateral, the centre
// of the incircle lies on the Newton line, the line joining the midpoints of
// the diagonals.
//
// The file records two results.  The first is the elementary Newton-line
// midpoint identity for an arbitrary quadrilateral: the segment joining the
// midpoints of the diagonals and the segment joining the midpoints of the
// opposite sides have a common midpoint, both being the average
// `(a + b + c + d) / 4` of the four vertices.  This identity is proved in
// full.  The second is the tangential statement itself, with the incircle
// tangency written through the squared distance to a side line:
// `dist(p, line(u, v))^2 = cross(p - u, v - u)^2 / |v - u|^2`, so the
// hypothesis `r^2 = dist(i, ab)^2` becomes
// `r^2 * |b - a|^2 = cross(i - a, b - a)^2`.  Its proof is commented out,
// with a sketch, because it needs the barycentric formula for the incenter
// of a tangential polygon and Pitot's theorem.

/// The pair version of pairing the two sums: (a + b) + (c + d) = (a + c) + (b + d).
theorem r2_add_pair_rearrange(
    a: Pair[Real, Real], b: Pair[Real, Real],
    c: Pair[Real, Real], d: Pair[Real, Real]
) {
    r2_add(r2_add(a, b), r2_add(c, d)) = r2_add(r2_add(a, c), r2_add(b, d))
} by {
    let lhs = r2_add(r2_add(a, b), r2_add(c, d))
    let rhs = r2_add(r2_add(a, c), r2_add(b, d))
    lhs.first = r2_add(a, b).first + r2_add(c, d).first
    r2_add(a, b).first = a.first + b.first
    r2_add(c, d).first = c.first + d.first
    lhs.first = (a.first + b.first) + (c.first + d.first)
    rhs.first = r2_add(a, c).first + r2_add(b, d).first
    r2_add(a, c).first = a.first + c.first
    r2_add(b, d).first = b.first + d.first
    rhs.first = (a.first + c.first) + (b.first + d.first)
    real_add_pair_rearrange(a.first, b.first, c.first, d.first)
    (a.first + b.first) + (c.first + d.first) = (a.first + c.first) + (b.first + d.first)
    lhs.first = rhs.first
    lhs.second = r2_add(a, b).second + r2_add(c, d).second
    r2_add(a, b).second = a.second + b.second
    r2_add(c, d).second = c.second + d.second
    lhs.second = (a.second + b.second) + (c.second + d.second)
    rhs.second = r2_add(a, c).second + r2_add(b, d).second
    r2_add(a, c).second = a.second + c.second
    r2_add(b, d).second = b.second + d.second
    rhs.second = (a.second + c.second) + (b.second + d.second)
    real_add_pair_rearrange(a.second, b.second, c.second, d.second)
    (a.second + b.second) + (c.second + d.second) = (a.second + c.second) + (b.second + d.second)
    lhs.second = rhs.second
    pair_ext(lhs, rhs)
}

/// The Newton-line midpoint identity: in any quadrilateral, the segment
/// joining the midpoints of the diagonals and the segment joining the
/// midpoints of the opposite sides have a common midpoint.
///
/// With `m1`, `m2` the midpoints of the diagonals `ac`, `bd` and `m3`, `m4`
/// the midpoints of the opposite sides `ab`, `cd`, the point `g` with
/// `g + g = m1 + m2` (the midpoint of the segment `m1 m2`) is also the
/// midpoint of the segment `m3 m4`: `g + g = m3 + m4`.
theorem theorems1000_newton_line_midpoint(
    a: Pair[Real, Real], b: Pair[Real, Real], c: Pair[Real, Real], d: Pair[Real, Real],
    m1: Pair[Real, Real], m2: Pair[Real, Real],
    m3: Pair[Real, Real], m4: Pair[Real, Real],
    g: Pair[Real, Real]
) {
    r2_add(m1, m1) = r2_add(a, c) and
    r2_add(m2, m2) = r2_add(b, d) and
    r2_add(m3, m3) = r2_add(a, b) and
    r2_add(m4, m4) = r2_add(c, d) and
    r2_add(g, g) = r2_add(m1, m2)
    implies
    r2_add(g, g) = r2_add(m3, m4)
} by {
    if r2_add(m1, m1) = r2_add(a, c) and
        r2_add(m2, m2) = r2_add(b, d) and
        r2_add(m3, m3) = r2_add(a, b) and
        r2_add(m4, m4) = r2_add(c, d) and
        r2_add(g, g) = r2_add(m1, m2) {
        r2_add(m1, m1) = r2_add(a, c)
        r2_add(m2, m2) = r2_add(b, d)
        r2_add(m3, m3) = r2_add(a, b)
        r2_add(m4, m4) = r2_add(c, d)
        r2_add(g, g) = r2_add(m1, m2)
        // 2(m1 + m2) = 2m1 + 2m2 = (a + c) + (b + d)
        r2_double_pair_rearrange(m1, m2)
        r2_add(r2_add(m1, m1), r2_add(m2, m2)) = r2_add(r2_add(m1, m2), r2_add(m1, m2))
        r2_add(r2_add(m1, m1), r2_add(m2, m2)) = r2_add(r2_add(a, c), r2_add(b, d))
        r2_add(r2_add(m1, m2), r2_add(m1, m2)) = r2_add(r2_add(a, c), r2_add(b, d))
        // 2g + 2g = 2(m1 + m2)
        r2_double_pair_rearrange(g, g)
        r2_add(r2_add(g, g), r2_add(g, g)) = r2_add(r2_add(g, g), r2_add(g, g))
        r2_add(r2_add(g, g), r2_add(g, g)) = r2_add(r2_add(m1, m2), r2_add(m1, m2))
        r2_add(r2_add(g, g), r2_add(g, g)) = r2_add(r2_add(a, c), r2_add(b, d))
        // 2(m3 + m4) = 2m3 + 2m4 = (a + b) + (c + d)
        r2_double_pair_rearrange(m3, m4)
        r2_add(r2_add(m3, m3), r2_add(m4, m4)) = r2_add(r2_add(m3, m4), r2_add(m3, m4))
        r2_add(r2_add(m3, m3), r2_add(m4, m4)) = r2_add(r2_add(a, b), r2_add(c, d))
        r2_add(r2_add(m3, m4), r2_add(m3, m4)) = r2_add(r2_add(a, b), r2_add(c, d))
        // (a + b) + (c + d) = (a + c) + (b + d)
        r2_add_pair_rearrange(a, b, c, d)
        r2_add(r2_add(a, b), r2_add(c, d)) = r2_add(r2_add(a, c), r2_add(b, d))
        r2_add(r2_add(m3, m4), r2_add(m3, m4)) = r2_add(r2_add(a, c), r2_add(b, d))
        r2_add(r2_add(g, g), r2_add(g, g)) = r2_add(r2_add(m3, m4), r2_add(m3, m4))
        // cancel the doubling
        r2_double_eq_double(r2_add(g, g), r2_add(m3, m4))
        r2_add(g, g) = r2_add(m3, m4)
    }
}

// Newton's theorem (quadrilateral): in a tangential quadrilateral the centre
// of the incircle lies on the Newton line, the line joining the midpoints of
// the diagonals.  The quadrilateral `a b c d` is tangential with incircle
// centre `i` and inradius `r`: the squared distance from `i` to each side
// line is `r^2`, i.e. `r^2 * |b-a|^2 = cross(i - a, b - a)^2` and cyclically.
// With `m1` the midpoint of `ac` and `m2` the midpoint of `bd`, the
// conclusion is the collinearity of `i`, `m1` and `m2`:
// `cross(i - m1, m2 - m1) = 0`.
//
// The statement is commented out because the proof needs the barycentric
// formula for the incenter of a tangential polygon (the incenter is the
// average of the vertices weighted by the side lengths), which itself
// requires the angle-bisector characterization of the incenter, and then a
// long cross-product expansion.  The sketch is recorded below.
//
// Proof sketch: write `p = |a-b|`, `q = |b-c|`, `t = |c-d|`, `s = |d-a|` for
// the side lengths and `P = p + q + t + s` for the perimeter.  In a
// tangential polygon the incenter is the weighted average of the vertices,
// `i = (p a + q b + t c + s d) / P`, and Pitot's theorem gives
// `p + t = q + s`.  The Newton line midpoint of `m1 m2` is
// `(a + b + c + d) / 4`, and the line through `m1` and `m2` consists of the
// points `m1 + u (m2 - m1)`.  Substituting the barycentric formula for `i`
// and expanding `cross(i - m1, m2 - m1)` with the cross-product bilinearity
// lemmas, all terms cancel exactly when `p + t = q + s`, which is Pitot's
// theorem; hence the cross product vanishes and `i` lies on the Newton line.
//
// theorem theorems1000_newton_quadrilateral(
//     a: Pair[Real, Real], b: Pair[Real, Real], c: Pair[Real, Real], d: Pair[Real, Real],
//     i: Pair[Real, Real], m1: Pair[Real, Real], m2: Pair[Real, Real],
//     r: Real
// ) {
//     r * r * r2_norm_sq(r2_sub(b, a)) = r2_cross(r2_sub(i, a), r2_sub(b, a)) * r2_cross(r2_sub(i, a), r2_sub(b, a)) and
//     r * r * r2_norm_sq(r2_sub(c, b)) = r2_cross(r2_sub(i, b), r2_sub(c, b)) * r2_cross(r2_sub(i, b), r2_sub(c, b)) and
//     r * r * r2_norm_sq(r2_sub(d, c)) = r2_cross(r2_sub(i, c), r2_sub(d, c)) * r2_cross(r2_sub(i, c), r2_sub(d, c)) and
//     r * r * r2_norm_sq(r2_sub(a, d)) = r2_cross(r2_sub(i, d), r2_sub(a, d)) * r2_cross(r2_sub(i, d), r2_sub(a, d)) and
//     r2_add(m1, m1) = r2_add(a, c) and
//     r2_add(m2, m2) = r2_add(b, d)
//     implies
//     r2_cross(r2_sub(i, m1), r2_sub(m2, m1)) = Real.0
// }
