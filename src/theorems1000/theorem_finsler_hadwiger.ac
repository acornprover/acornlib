from pair import Pair
from real import Real
from top100 import r2_add, r2_neg, r2_sub, r2_dot, r2_norm_sq
from theorems1000.r2_helpers import r2_rot90, r2_rot90_def, r2_rot90_first, r2_rot90_second,
    r2_rot90_add, r2_rot90_sub, r2_rot90_smul, r2_rot90_norm_sq, r2_dot_rot90_self,
    r2_dot_rot90_comm, r2_dot_rot90_swap, r2_rot90_twice, r2_double_sub_distrib,
    r2_sub_add_pair_distrib, r2_add_comm, r2_add_assoc, r2_sub_reverse_neg,
    r2_double_eq_double, r2_dot_comm, r2_sub_sub_same_base, r2_add_zero_left

// The Finsler-Hadwiger theorem: given two squares `ABCD` and `AB'C'D'`
// sharing the vertex `a`, let `f` and `h` be the centers of the two squares
// and let `e` and `g` be the midpoints of the segments `B'D` and `D'B`
// respectively (the "cross" segments joining far vertices).  Then the
// quadrilateral `efgh` is a square: each side has the same length and each
// pair of adjacent sides is perpendicular.
//
// With `u = b - a` and `v = b' - a` the two side vectors, the four points
// satisfy `2f = a + b + R90(u)`, `2h = a + b' + R90(v)`,
// `2e = b' + a + R90(u)` (midpoint of `b'` and `d = a + R90(u)`) and
// `2g = a + R90(v) + b` (midpoint of `d' = a + R90(v)` and `b`), where
// `R90` is the quarter turn.  The doubled displacements between consecutive
// points are then `2(e - f) = b' - b`, `2(f - g) = R90(b - b')` and
// `2(g - h) = b - b'`, so that `e - f = R90(f - g)` and cyclically: each
// side of `efgh` is the quarter turn of the next, which is exactly the
// defining property of a square.  The displacement identities are proved
// below and the square property follows from the quarter-turn lemmas.

/// The doubled displacement from `e` to `f` is the difference of the two
/// square side vectors: `2(e - f) = b' - b`.
theorem finsler_e_minus_f(
    a: Pair[Real, Real], b: Pair[Real, Real], bp: Pair[Real, Real],
    e: Pair[Real, Real], f: Pair[Real, Real]
) {
    r2_add(e, e) = r2_add(r2_add(bp, a), r2_rot90(r2_sub(b, a))) and
    r2_add(f, f) = r2_add(r2_add(a, b), r2_rot90(r2_sub(b, a)))
    implies
    r2_add(r2_sub(e, f), r2_sub(e, f)) = r2_sub(bp, b)
} by {
    if r2_add(e, e) = r2_add(r2_add(bp, a), r2_rot90(r2_sub(b, a))) and
        r2_add(f, f) = r2_add(r2_add(a, b), r2_rot90(r2_sub(b, a))) {
        r2_add(e, e) = r2_add(r2_add(bp, a), r2_rot90(r2_sub(b, a)))
        r2_add(f, f) = r2_add(r2_add(a, b), r2_rot90(r2_sub(b, a)))
        r2_double_sub_distrib(e, f)
        r2_add(r2_sub(e, f), r2_sub(e, f)) = r2_sub(r2_add(e, e), r2_add(f, f))
        r2_sub(r2_add(e, e), r2_add(f, f)) =
            r2_sub(
                r2_add(r2_add(bp, a), r2_rot90(r2_sub(b, a))),
                r2_add(r2_add(a, b), r2_rot90(r2_sub(b, a))))
        r2_sub_add_pair_distrib(r2_add(bp, a), r2_add(a, b), r2_rot90(r2_sub(b, a)), r2_rot90(r2_sub(b, a)))
        r2_sub(
            r2_add(r2_add(bp, a), r2_rot90(r2_sub(b, a))),
            r2_add(r2_add(a, b), r2_rot90(r2_sub(b, a)))) =
            r2_add(r2_sub(r2_add(bp, a), r2_add(a, b)), r2_sub(r2_rot90(r2_sub(b, a)), r2_rot90(r2_sub(b, a))))
        r2_sub(r2_rot90(r2_sub(b, a)), r2_rot90(r2_sub(b, a))) = Pair.new(Real.0, Real.0)
        r2_add(r2_sub(r2_add(bp, a), r2_add(a, b)), r2_sub(r2_rot90(r2_sub(b, a)), r2_rot90(r2_sub(b, a)))) =
            r2_sub(r2_add(bp, a), r2_add(a, b))
        r2_sub(r2_add(e, e), r2_add(f, f)) = r2_sub(r2_add(bp, a), r2_add(a, b))
        r2_sub_add_pair_distrib(bp, a, a, b)
        r2_sub(r2_add(bp, a), r2_add(a, b)) = r2_add(r2_sub(bp, a), r2_sub(a, b))
        r2_add(r2_sub(bp, a), r2_sub(a, b)) = r2_sub(bp, b)
        r2_sub(r2_add(bp, a), r2_add(a, b)) = r2_sub(bp, b)
        r2_sub(r2_add(e, e), r2_add(f, f)) = r2_sub(bp, b)
        r2_add(r2_sub(e, f), r2_sub(e, f)) = r2_sub(bp, b)
    }
}

/// The doubled displacement from `f` to `g` is the quarter turn of the
/// reversed side difference: `2(f - g) = R90(b - b')`.
theorem finsler_f_minus_g(
    a: Pair[Real, Real], b: Pair[Real, Real], bp: Pair[Real, Real],
    f: Pair[Real, Real], g: Pair[Real, Real]
) {
    r2_add(f, f) = r2_add(r2_add(a, b), r2_rot90(r2_sub(b, a))) and
    r2_add(g, g) = r2_add(r2_add(a, r2_rot90(r2_sub(bp, a))), b)
    implies
    r2_add(r2_sub(f, g), r2_sub(f, g)) = r2_rot90(r2_sub(b, bp))
} by {
    if r2_add(f, f) = r2_add(r2_add(a, b), r2_rot90(r2_sub(b, a))) and
        r2_add(g, g) = r2_add(r2_add(a, r2_rot90(r2_sub(bp, a))), b) {
        r2_add(f, f) = r2_add(r2_add(a, b), r2_rot90(r2_sub(b, a)))
        r2_add(g, g) = r2_add(r2_add(a, r2_rot90(r2_sub(bp, a))), b)
        // 2g = (a + b) + R90(b' - a): reassociate the sum.
        r2_add_assoc(a, r2_rot90(r2_sub(bp, a)), b)
        r2_add(r2_add(a, r2_rot90(r2_sub(bp, a))), b) = r2_add(a, r2_add(r2_rot90(r2_sub(bp, a)), b))
        r2_add_comm(r2_rot90(r2_sub(bp, a)), b)
        r2_add(r2_rot90(r2_sub(bp, a)), b) = r2_add(b, r2_rot90(r2_sub(bp, a)))
        r2_add(a, r2_add(r2_rot90(r2_sub(bp, a)), b)) = r2_add(a, r2_add(b, r2_rot90(r2_sub(bp, a))))
        r2_add_assoc(a, b, r2_rot90(r2_sub(bp, a)))
        r2_add(r2_add(a, b), r2_rot90(r2_sub(bp, a))) = r2_add(a, r2_add(b, r2_rot90(r2_sub(bp, a))))
        r2_add(r2_add(a, r2_rot90(r2_sub(bp, a))), b) = r2_add(r2_add(a, b), r2_rot90(r2_sub(bp, a)))
        r2_add(g, g) = r2_add(r2_add(a, b), r2_rot90(r2_sub(bp, a)))
        r2_double_sub_distrib(f, g)
        r2_add(r2_sub(f, g), r2_sub(f, g)) = r2_sub(r2_add(f, f), r2_add(g, g))
        r2_sub(r2_add(f, f), r2_add(g, g)) =
            r2_sub(
                r2_add(r2_add(a, b), r2_rot90(r2_sub(b, a))),
                r2_add(r2_add(a, b), r2_rot90(r2_sub(bp, a))))
        r2_sub_add_pair_distrib(r2_add(a, b), r2_add(a, b), r2_rot90(r2_sub(b, a)), r2_rot90(r2_sub(bp, a)))
        r2_sub(
            r2_add(r2_add(a, b), r2_rot90(r2_sub(b, a))),
            r2_add(r2_add(a, b), r2_rot90(r2_sub(bp, a)))) =
            r2_add(
                r2_sub(r2_add(a, b), r2_add(a, b)),
                r2_sub(r2_rot90(r2_sub(b, a)), r2_rot90(r2_sub(bp, a))))
        r2_sub(r2_add(a, b), r2_add(a, b)) = Pair.new(Real.0, Real.0)
        r2_add_zero_left(r2_sub(r2_rot90(r2_sub(b, a)), r2_rot90(r2_sub(bp, a))))
        r2_add(
            r2_sub(r2_add(a, b), r2_add(a, b)),
            r2_sub(r2_rot90(r2_sub(b, a)), r2_rot90(r2_sub(bp, a)))) =
            r2_sub(r2_rot90(r2_sub(b, a)), r2_rot90(r2_sub(bp, a)))
        r2_sub(r2_add(f, f), r2_add(g, g)) =
            r2_sub(r2_rot90(r2_sub(b, a)), r2_rot90(r2_sub(bp, a)))
        // R90(b - a) - R90(b' - a) = R90(b - b')
        r2_rot90_sub(b, a)
        r2_rot90(r2_sub(b, a)) = r2_sub(r2_rot90(b), r2_rot90(a))
        r2_rot90_sub(bp, a)
        r2_rot90(r2_sub(bp, a)) = r2_sub(r2_rot90(bp), r2_rot90(a))
        r2_sub(r2_rot90(r2_sub(b, a)), r2_rot90(r2_sub(bp, a))) =
            r2_sub(r2_sub(r2_rot90(b), r2_rot90(a)), r2_sub(r2_rot90(bp), r2_rot90(a)))
        r2_sub_sub_same_base(r2_rot90(a), r2_rot90(b), r2_rot90(bp))
        r2_sub(r2_sub(r2_rot90(b), r2_rot90(a)), r2_sub(r2_rot90(bp), r2_rot90(a))) =
            r2_sub(r2_rot90(b), r2_rot90(bp))
        r2_sub(r2_rot90(r2_sub(b, a)), r2_rot90(r2_sub(bp, a))) = r2_sub(r2_rot90(b), r2_rot90(bp))
        r2_rot90_sub(b, bp)
        r2_rot90(r2_sub(b, bp)) = r2_sub(r2_rot90(b), r2_rot90(bp))
        r2_sub(r2_rot90(r2_sub(b, a)), r2_rot90(r2_sub(bp, a))) = r2_rot90(r2_sub(b, bp))
        r2_sub(r2_add(f, f), r2_add(g, g)) = r2_rot90(r2_sub(b, bp))
        r2_add(r2_sub(f, g), r2_sub(f, g)) = r2_rot90(r2_sub(b, bp))
    }
}

/// The doubled displacement from `g` to `h` is the reversed side difference:
/// `2(g - h) = b - b'`.
theorem finsler_g_minus_h(
    a: Pair[Real, Real], b: Pair[Real, Real], bp: Pair[Real, Real],
    g: Pair[Real, Real], h: Pair[Real, Real]
) {
    r2_add(g, g) = r2_add(r2_add(a, r2_rot90(r2_sub(bp, a))), b) and
    r2_add(h, h) = r2_add(r2_add(a, bp), r2_rot90(r2_sub(bp, a)))
    implies
    r2_add(r2_sub(g, h), r2_sub(g, h)) = r2_sub(b, bp)
} by {
    if r2_add(g, g) = r2_add(r2_add(a, r2_rot90(r2_sub(bp, a))), b) and
        r2_add(h, h) = r2_add(r2_add(a, bp), r2_rot90(r2_sub(bp, a))) {
        r2_add(g, g) = r2_add(r2_add(a, r2_rot90(r2_sub(bp, a))), b)
        r2_add(h, h) = r2_add(r2_add(a, bp), r2_rot90(r2_sub(bp, a)))
        r2_add_assoc(a, r2_rot90(r2_sub(bp, a)), b)
        r2_add(r2_add(a, r2_rot90(r2_sub(bp, a))), b) = r2_add(a, r2_add(r2_rot90(r2_sub(bp, a)), b))
        r2_add_comm(r2_rot90(r2_sub(bp, a)), b)
        r2_add(r2_rot90(r2_sub(bp, a)), b) = r2_add(b, r2_rot90(r2_sub(bp, a)))
        r2_add(a, r2_add(r2_rot90(r2_sub(bp, a)), b)) = r2_add(a, r2_add(b, r2_rot90(r2_sub(bp, a))))
        r2_add_assoc(a, b, r2_rot90(r2_sub(bp, a)))
        r2_add(r2_add(a, b), r2_rot90(r2_sub(bp, a))) = r2_add(a, r2_add(b, r2_rot90(r2_sub(bp, a))))
        r2_add(r2_add(a, r2_rot90(r2_sub(bp, a))), b) = r2_add(r2_add(a, b), r2_rot90(r2_sub(bp, a)))
        r2_add(g, g) = r2_add(r2_add(a, b), r2_rot90(r2_sub(bp, a)))
        r2_double_sub_distrib(g, h)
        r2_add(r2_sub(g, h), r2_sub(g, h)) = r2_sub(r2_add(g, g), r2_add(h, h))
        r2_sub(r2_add(g, g), r2_add(h, h)) =
            r2_sub(
                r2_add(r2_add(a, b), r2_rot90(r2_sub(bp, a))),
                r2_add(r2_add(a, bp), r2_rot90(r2_sub(bp, a))))
        r2_sub_add_pair_distrib(r2_add(a, b), r2_add(a, bp), r2_rot90(r2_sub(bp, a)), r2_rot90(r2_sub(bp, a)))
        r2_sub(
            r2_add(r2_add(a, b), r2_rot90(r2_sub(bp, a))),
            r2_add(r2_add(a, bp), r2_rot90(r2_sub(bp, a)))) =
            r2_add(
                r2_sub(r2_add(a, b), r2_add(a, bp)),
                r2_sub(r2_rot90(r2_sub(bp, a)), r2_rot90(r2_sub(bp, a))))
        r2_sub(r2_rot90(r2_sub(bp, a)), r2_rot90(r2_sub(bp, a))) = Pair.new(Real.0, Real.0)
        r2_add(
            r2_sub(r2_add(a, b), r2_add(a, bp)),
            r2_sub(r2_rot90(r2_sub(bp, a)), r2_rot90(r2_sub(bp, a)))) =
            r2_sub(r2_add(a, b), r2_add(a, bp))
        r2_sub(r2_add(g, g), r2_add(h, h)) = r2_sub(r2_add(a, b), r2_add(a, bp))
        r2_sub_add_pair_distrib(a, a, b, bp)
        r2_sub(r2_add(a, b), r2_add(a, bp)) = r2_add(r2_sub(a, a), r2_sub(b, bp))
        r2_sub(a, a) = Pair.new(Real.0, Real.0)
        r2_add_zero_left(r2_sub(b, bp))
        r2_add(r2_sub(a, a), r2_sub(b, bp)) = r2_sub(b, bp)
        r2_sub(r2_add(a, b), r2_add(a, bp)) = r2_sub(b, bp)
        r2_sub(r2_add(g, g), r2_add(h, h)) = r2_sub(b, bp)
        r2_add(r2_sub(g, h), r2_sub(g, h)) = r2_sub(b, bp)
    }
}

/// The doubled displacement from `h` to `e` is the quarter turn of the side
/// difference: `2(h - e) = R90(b' - b)`.
theorem finsler_h_minus_e(
    a: Pair[Real, Real], b: Pair[Real, Real], bp: Pair[Real, Real],
    h: Pair[Real, Real], e: Pair[Real, Real]
) {
    r2_add(h, h) = r2_add(r2_add(a, bp), r2_rot90(r2_sub(bp, a))) and
    r2_add(e, e) = r2_add(r2_add(bp, a), r2_rot90(r2_sub(b, a)))
    implies
    r2_add(r2_sub(h, e), r2_sub(h, e)) = r2_rot90(r2_sub(bp, b))
} by {
    if r2_add(h, h) = r2_add(r2_add(a, bp), r2_rot90(r2_sub(bp, a))) and
        r2_add(e, e) = r2_add(r2_add(bp, a), r2_rot90(r2_sub(b, a))) {
        r2_add(h, h) = r2_add(r2_add(a, bp), r2_rot90(r2_sub(bp, a)))
        r2_add(e, e) = r2_add(r2_add(bp, a), r2_rot90(r2_sub(b, a)))
        r2_add_comm(bp, a)
        r2_add(bp, a) = r2_add(a, bp)
        r2_add(e, e) = r2_add(r2_add(a, bp), r2_rot90(r2_sub(b, a)))
        r2_double_sub_distrib(h, e)
        r2_add(r2_sub(h, e), r2_sub(h, e)) = r2_sub(r2_add(h, h), r2_add(e, e))
        r2_sub(r2_add(h, h), r2_add(e, e)) =
            r2_sub(
                r2_add(r2_add(a, bp), r2_rot90(r2_sub(bp, a))),
                r2_add(r2_add(a, bp), r2_rot90(r2_sub(b, a))))
        r2_sub_add_pair_distrib(r2_add(a, bp), r2_add(a, bp), r2_rot90(r2_sub(bp, a)), r2_rot90(r2_sub(b, a)))
        r2_sub(
            r2_add(r2_add(a, bp), r2_rot90(r2_sub(bp, a))),
            r2_add(r2_add(a, bp), r2_rot90(r2_sub(b, a)))) =
            r2_add(
                r2_sub(r2_add(a, bp), r2_add(a, bp)),
                r2_sub(r2_rot90(r2_sub(bp, a)), r2_rot90(r2_sub(b, a))))
        r2_sub(r2_add(a, bp), r2_add(a, bp)) = Pair.new(Real.0, Real.0)
        r2_add_zero_left(r2_sub(r2_rot90(r2_sub(bp, a)), r2_rot90(r2_sub(b, a))))
        r2_add(
            r2_sub(r2_add(a, bp), r2_add(a, bp)),
            r2_sub(r2_rot90(r2_sub(bp, a)), r2_rot90(r2_sub(b, a)))) =
            r2_sub(r2_rot90(r2_sub(bp, a)), r2_rot90(r2_sub(b, a)))
        r2_sub(r2_add(h, h), r2_add(e, e)) =
            r2_sub(r2_rot90(r2_sub(bp, a)), r2_rot90(r2_sub(b, a)))
        r2_rot90_sub(bp, a)
        r2_rot90(r2_sub(bp, a)) = r2_sub(r2_rot90(bp), r2_rot90(a))
        r2_rot90_sub(b, a)
        r2_rot90(r2_sub(b, a)) = r2_sub(r2_rot90(b), r2_rot90(a))
        r2_sub(r2_rot90(r2_sub(bp, a)), r2_rot90(r2_sub(b, a))) =
            r2_sub(r2_sub(r2_rot90(bp), r2_rot90(a)), r2_sub(r2_rot90(b), r2_rot90(a)))
        r2_sub_sub_same_base(r2_rot90(a), r2_rot90(bp), r2_rot90(b))
        r2_sub(r2_sub(r2_rot90(bp), r2_rot90(a)), r2_sub(r2_rot90(b), r2_rot90(a))) =
            r2_sub(r2_rot90(bp), r2_rot90(b))
        r2_sub(r2_rot90(r2_sub(bp, a)), r2_rot90(r2_sub(b, a))) = r2_sub(r2_rot90(bp), r2_rot90(b))
        r2_rot90_sub(bp, b)
        r2_rot90(r2_sub(bp, b)) = r2_sub(r2_rot90(bp), r2_rot90(b))
        r2_sub(r2_rot90(r2_sub(bp, a)), r2_rot90(r2_sub(b, a))) = r2_rot90(r2_sub(bp, b))
        r2_sub(r2_add(h, h), r2_add(e, e)) = r2_rot90(r2_sub(bp, b))
        r2_add(r2_sub(h, e), r2_sub(h, e)) = r2_rot90(r2_sub(bp, b))
    }
}

/// Each side of the Finsler-Hadwiger quadrilateral is the quarter turn of
/// the next: `e - f = R90(f - g)`.
theorem finsler_ef_rot90_fg(
    a: Pair[Real, Real], b: Pair[Real, Real], bp: Pair[Real, Real],
    e: Pair[Real, Real], f: Pair[Real, Real], g: Pair[Real, Real]
) {
    r2_add(e, e) = r2_add(r2_add(bp, a), r2_rot90(r2_sub(b, a))) and
    r2_add(f, f) = r2_add(r2_add(a, b), r2_rot90(r2_sub(b, a))) and
    r2_add(g, g) = r2_add(r2_add(a, r2_rot90(r2_sub(bp, a))), b)
    implies
    r2_sub(e, f) = r2_rot90(r2_sub(f, g))
} by {
    if r2_add(e, e) = r2_add(r2_add(bp, a), r2_rot90(r2_sub(b, a))) and
        r2_add(f, f) = r2_add(r2_add(a, b), r2_rot90(r2_sub(b, a))) and
        r2_add(g, g) = r2_add(r2_add(a, r2_rot90(r2_sub(bp, a))), b) {
        finsler_e_minus_f(a, b, bp, e, f)
        r2_add(r2_sub(e, f), r2_sub(e, f)) = r2_sub(bp, b)
        finsler_f_minus_g(a, b, bp, f, g)
        r2_add(r2_sub(f, g), r2_sub(f, g)) = r2_rot90(r2_sub(b, bp))
        r2_rot90_add(r2_sub(f, g), r2_sub(f, g))
        r2_rot90(r2_add(r2_sub(f, g), r2_sub(f, g))) = r2_add(r2_rot90(r2_sub(f, g)), r2_rot90(r2_sub(f, g)))
        r2_rot90(r2_add(r2_sub(f, g), r2_sub(f, g))) = r2_rot90(r2_rot90(r2_sub(b, bp)))
        r2_rot90_twice(r2_sub(b, bp))
        r2_rot90(r2_rot90(r2_sub(b, bp))) = r2_neg(r2_sub(b, bp))
        r2_sub_reverse_neg(bp, b)
        r2_sub(bp, b) = r2_neg(r2_sub(b, bp))
        r2_neg(r2_sub(b, bp)) = r2_sub(bp, b)
        r2_rot90(r2_rot90(r2_sub(b, bp))) = r2_sub(bp, b)
        r2_rot90(r2_add(r2_sub(f, g), r2_sub(f, g))) = r2_sub(bp, b)
        r2_add(r2_rot90(r2_sub(f, g)), r2_rot90(r2_sub(f, g))) = r2_sub(bp, b)
        r2_add(r2_sub(e, f), r2_sub(e, f)) = r2_add(r2_rot90(r2_sub(f, g)), r2_rot90(r2_sub(f, g)))
        r2_double_eq_double(r2_sub(e, f), r2_rot90(r2_sub(f, g)))
        r2_sub(e, f) = r2_rot90(r2_sub(f, g))
    }
}

/// The second side identity: `f - g = R90(g - h)`.
theorem finsler_fg_rot90_gh(
    a: Pair[Real, Real], b: Pair[Real, Real], bp: Pair[Real, Real],
    f: Pair[Real, Real], g: Pair[Real, Real], h: Pair[Real, Real]
) {
    r2_add(f, f) = r2_add(r2_add(a, b), r2_rot90(r2_sub(b, a))) and
    r2_add(g, g) = r2_add(r2_add(a, r2_rot90(r2_sub(bp, a))), b) and
    r2_add(h, h) = r2_add(r2_add(a, bp), r2_rot90(r2_sub(bp, a)))
    implies
    r2_sub(f, g) = r2_rot90(r2_sub(g, h))
} by {
    if r2_add(f, f) = r2_add(r2_add(a, b), r2_rot90(r2_sub(b, a))) and
        r2_add(g, g) = r2_add(r2_add(a, r2_rot90(r2_sub(bp, a))), b) and
        r2_add(h, h) = r2_add(r2_add(a, bp), r2_rot90(r2_sub(bp, a))) {
        finsler_f_minus_g(a, b, bp, f, g)
        r2_add(r2_sub(f, g), r2_sub(f, g)) = r2_rot90(r2_sub(b, bp))
        finsler_g_minus_h(a, b, bp, g, h)
        r2_add(r2_sub(g, h), r2_sub(g, h)) = r2_sub(b, bp)
        r2_rot90_add(r2_sub(g, h), r2_sub(g, h))
        r2_rot90(r2_add(r2_sub(g, h), r2_sub(g, h))) = r2_add(r2_rot90(r2_sub(g, h)), r2_rot90(r2_sub(g, h)))
        r2_rot90(r2_add(r2_sub(g, h), r2_sub(g, h))) = r2_rot90(r2_sub(b, bp))
        r2_add(r2_rot90(r2_sub(g, h)), r2_rot90(r2_sub(g, h))) = r2_rot90(r2_sub(b, bp))
        r2_add(r2_sub(f, g), r2_sub(f, g)) = r2_add(r2_rot90(r2_sub(g, h)), r2_rot90(r2_sub(g, h)))
        r2_double_eq_double(r2_sub(f, g), r2_rot90(r2_sub(g, h)))
        r2_sub(f, g) = r2_rot90(r2_sub(g, h))
    }
}

/// The third side identity: `g - h = R90(h - e)`.
theorem finsler_gh_rot90_he(
    a: Pair[Real, Real], b: Pair[Real, Real], bp: Pair[Real, Real],
    g: Pair[Real, Real], h: Pair[Real, Real], e: Pair[Real, Real]
) {
    r2_add(g, g) = r2_add(r2_add(a, r2_rot90(r2_sub(bp, a))), b) and
    r2_add(h, h) = r2_add(r2_add(a, bp), r2_rot90(r2_sub(bp, a))) and
    r2_add(e, e) = r2_add(r2_add(bp, a), r2_rot90(r2_sub(b, a)))
    implies
    r2_sub(g, h) = r2_rot90(r2_sub(h, e))
} by {
    if r2_add(g, g) = r2_add(r2_add(a, r2_rot90(r2_sub(bp, a))), b) and
        r2_add(h, h) = r2_add(r2_add(a, bp), r2_rot90(r2_sub(bp, a))) and
        r2_add(e, e) = r2_add(r2_add(bp, a), r2_rot90(r2_sub(b, a))) {
        finsler_g_minus_h(a, b, bp, g, h)
        r2_add(r2_sub(g, h), r2_sub(g, h)) = r2_sub(b, bp)
        finsler_h_minus_e(a, b, bp, h, e)
        r2_add(r2_sub(h, e), r2_sub(h, e)) = r2_rot90(r2_sub(bp, b))
        r2_rot90_add(r2_sub(h, e), r2_sub(h, e))
        r2_rot90(r2_add(r2_sub(h, e), r2_sub(h, e))) = r2_add(r2_rot90(r2_sub(h, e)), r2_rot90(r2_sub(h, e)))
        r2_rot90(r2_add(r2_sub(h, e), r2_sub(h, e))) = r2_rot90(r2_rot90(r2_sub(bp, b)))
        r2_rot90_twice(r2_sub(bp, b))
        r2_rot90(r2_rot90(r2_sub(bp, b))) = r2_neg(r2_sub(bp, b))
        r2_sub_reverse_neg(b, bp)
        r2_sub(b, bp) = r2_neg(r2_sub(bp, b))
        r2_neg(r2_sub(bp, b)) = r2_sub(b, bp)
        r2_rot90(r2_rot90(r2_sub(bp, b))) = r2_sub(b, bp)
        r2_rot90(r2_add(r2_sub(h, e), r2_sub(h, e))) = r2_sub(b, bp)
        r2_add(r2_rot90(r2_sub(h, e)), r2_rot90(r2_sub(h, e))) = r2_sub(b, bp)
        r2_add(r2_sub(g, h), r2_sub(g, h)) = r2_add(r2_rot90(r2_sub(h, e)), r2_rot90(r2_sub(h, e)))
        r2_double_eq_double(r2_sub(g, h), r2_rot90(r2_sub(h, e)))
        r2_sub(g, h) = r2_rot90(r2_sub(h, e))
    }
}

/// The Finsler-Hadwiger theorem.
///
/// Two squares `ABCD` and `AB'C'D'` share the vertex `a`, with `f` and `h`
/// their centers and `e`, `g` the midpoints of the segments `B'D` and `D'B`.
/// The quadrilateral `efgh` is a square: the four sides have equal lengths
/// and each pair of adjacent sides is perpendicular.
theorem theorems1000_finsler_hadwiger(
    a: Pair[Real, Real], b: Pair[Real, Real], bp: Pair[Real, Real],
    e: Pair[Real, Real], f: Pair[Real, Real], g: Pair[Real, Real], h: Pair[Real, Real]
) {
    r2_add(f, f) = r2_add(r2_add(a, b), r2_rot90(r2_sub(b, a))) and
    r2_add(h, h) = r2_add(r2_add(a, bp), r2_rot90(r2_sub(bp, a))) and
    r2_add(e, e) = r2_add(r2_add(bp, a), r2_rot90(r2_sub(b, a))) and
    r2_add(g, g) = r2_add(r2_add(a, r2_rot90(r2_sub(bp, a))), b)
    implies
    r2_norm_sq(r2_sub(e, f)) = r2_norm_sq(r2_sub(f, g)) and
    r2_norm_sq(r2_sub(f, g)) = r2_norm_sq(r2_sub(g, h)) and
    r2_norm_sq(r2_sub(g, h)) = r2_norm_sq(r2_sub(h, e)) and
    r2_dot(r2_sub(e, f), r2_sub(f, g)) = Real.0 and
    r2_dot(r2_sub(f, g), r2_sub(g, h)) = Real.0 and
    r2_dot(r2_sub(g, h), r2_sub(h, e)) = Real.0
} by {
    if r2_add(f, f) = r2_add(r2_add(a, b), r2_rot90(r2_sub(b, a))) and
        r2_add(h, h) = r2_add(r2_add(a, bp), r2_rot90(r2_sub(bp, a))) and
        r2_add(e, e) = r2_add(r2_add(bp, a), r2_rot90(r2_sub(b, a))) and
        r2_add(g, g) = r2_add(r2_add(a, r2_rot90(r2_sub(bp, a))), b) {
        r2_add(f, f) = r2_add(r2_add(a, b), r2_rot90(r2_sub(b, a)))
        r2_add(h, h) = r2_add(r2_add(a, bp), r2_rot90(r2_sub(bp, a)))
        r2_add(e, e) = r2_add(r2_add(bp, a), r2_rot90(r2_sub(b, a)))
        r2_add(g, g) = r2_add(r2_add(a, r2_rot90(r2_sub(bp, a))), b)
        // side ef: each side is the quarter turn of the next.
        finsler_ef_rot90_fg(a, b, bp, e, f, g)
        r2_sub(e, f) = r2_rot90(r2_sub(f, g))
        r2_rot90_norm_sq(r2_sub(f, g))
        r2_norm_sq(r2_rot90(r2_sub(f, g))) = r2_norm_sq(r2_sub(f, g))
        r2_norm_sq(r2_sub(e, f)) = r2_norm_sq(r2_sub(f, g))
        r2_dot_rot90_self(r2_sub(f, g))
        r2_dot(r2_sub(f, g), r2_rot90(r2_sub(f, g))) = Real.0
        r2_dot_comm(r2_rot90(r2_sub(f, g)), r2_sub(f, g))
        r2_dot(r2_rot90(r2_sub(f, g)), r2_sub(f, g)) = r2_dot(r2_sub(f, g), r2_rot90(r2_sub(f, g)))
        r2_dot(r2_sub(e, f), r2_sub(f, g)) = Real.0
        // side fg
        finsler_fg_rot90_gh(a, b, bp, f, g, h)
        r2_sub(f, g) = r2_rot90(r2_sub(g, h))
        r2_rot90_norm_sq(r2_sub(g, h))
        r2_norm_sq(r2_rot90(r2_sub(g, h))) = r2_norm_sq(r2_sub(g, h))
        r2_norm_sq(r2_sub(f, g)) = r2_norm_sq(r2_sub(g, h))
        r2_dot_rot90_self(r2_sub(g, h))
        r2_dot(r2_sub(g, h), r2_rot90(r2_sub(g, h))) = Real.0
        r2_dot_comm(r2_rot90(r2_sub(g, h)), r2_sub(g, h))
        r2_dot(r2_rot90(r2_sub(g, h)), r2_sub(g, h)) = r2_dot(r2_sub(g, h), r2_rot90(r2_sub(g, h)))
        r2_dot(r2_sub(f, g), r2_sub(g, h)) = Real.0
        // side gh
        finsler_gh_rot90_he(a, b, bp, g, h, e)
        r2_sub(g, h) = r2_rot90(r2_sub(h, e))
        r2_rot90_norm_sq(r2_sub(h, e))
        r2_norm_sq(r2_rot90(r2_sub(h, e))) = r2_norm_sq(r2_sub(h, e))
        r2_norm_sq(r2_sub(g, h)) = r2_norm_sq(r2_sub(h, e))
        r2_dot_rot90_self(r2_sub(h, e))
        r2_dot(r2_sub(h, e), r2_rot90(r2_sub(h, e))) = Real.0
        r2_dot_comm(r2_rot90(r2_sub(h, e)), r2_sub(h, e))
        r2_dot(r2_rot90(r2_sub(h, e)), r2_sub(h, e)) = r2_dot(r2_sub(h, e), r2_rot90(r2_sub(h, e)))
        r2_dot(r2_sub(g, h), r2_sub(h, e)) = Real.0
        r2_norm_sq(r2_sub(e, f)) = r2_norm_sq(r2_sub(f, g)) and
            r2_norm_sq(r2_sub(f, g)) = r2_norm_sq(r2_sub(g, h)) and
            r2_norm_sq(r2_sub(g, h)) = r2_norm_sq(r2_sub(h, e)) and
            r2_dot(r2_sub(e, f), r2_sub(f, g)) = Real.0 and
            r2_dot(r2_sub(f, g), r2_sub(g, h)) = Real.0 and
            r2_dot(r2_sub(g, h), r2_sub(h, e)) = Real.0
    }
}
