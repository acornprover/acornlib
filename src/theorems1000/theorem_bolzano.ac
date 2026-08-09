from real import Real, continuous, intermediate_value_closed_interval
from order_set import closed_interval_set, closed_interval_set_lower_le,
    closed_interval_set_le_upper

numerals Real

// Bolzano's theorem (Bernard Bolzano, 1817), also called the
// intermediate-zero theorem: a continuous real function that takes values
// of opposite signs at the endpoints of an interval must vanish somewhere
// inside it.  Bolzano proved it as a lemma toward the intermediate value
// theorem, which states the same conclusion for an arbitrary intermediate
// value `t` instead of zero; conversely, the intermediate value theorem
// follows from Bolzano's theorem by applying it to `f - t`.  Bolzano's
// proof, with its explicit appeal to completeness, is often regarded as the
// first rigorous argument of real analysis; Cauchy's 1821 exposition made
// the theorem standard.
//
// The first statement below is the theorem for sign changes that may touch
// zero at an endpoint: if `f(a) <= 0 <= f(b)`, there is a zero `c` in
// `[a, b]`.  The second is the classical strict form: if
// `f(a) < 0 < f(b)`, the zero lies strictly inside the interval,
// `a < c < b`.
//
// Proof: both statements are immediate consequences of the library's
// intermediate value theorem `intermediate_value_closed_interval`, applied
// with the target value `Real.0`; the strict form additionally rules out
// the endpoints by the strict signs there.

/// Bolzano's theorem: a continuous function taking values of opposite
/// (non-strict) signs at the endpoints of an interval has a zero in the
/// interval.
theorem theorems1000_bolzano(f: Real -> Real, a: Real, b: Real) {
    continuous(f) and a <= b and f(a) <= Real.0 and Real.0 <= f(b)
    implies exists(c: Real) { a <= c and c <= b and f(c) = Real.0 }
} by {
    if continuous(f) and a <= b and f(a) <= Real.0 and Real.0 <= f(b) {
        intermediate_value_closed_interval(f, a, b, Real.0)
        exists(point: Real) {
            closed_interval_set(a, b).contains(point) and f(point) = Real.0
        }
        let point: Real satisfy {
            closed_interval_set(a, b).contains(point) and f(point) = Real.0
        }
        closed_interval_set_lower_le(a, b, point)
        a <= point
        closed_interval_set_le_upper(a, b, point)
        point <= b
        exists(c: Real) { a <= c and c <= b and f(c) = Real.0 }
    }
}

/// Bolzano's theorem, strict form: opposite strict signs at the endpoints
/// force a zero strictly inside the interval.
theorem theorems1000_bolzano_strict(f: Real -> Real, a: Real, b: Real) {
    continuous(f) and a < b and f(a) < Real.0 and Real.0 < f(b)
    implies exists(c: Real) { a < c and c < b and f(c) = Real.0 }
} by {
    if continuous(f) and a < b and f(a) < Real.0 and Real.0 < f(b) {
        a <= b
        f(a) <= Real.0
        Real.0 <= f(b)
        theorems1000_bolzano(f, a, b)
        exists(c: Real) { a <= c and c <= b and f(c) = Real.0 }
        let c: Real satisfy {
            a <= c and c <= b and f(c) = Real.0
        }
        f(c) = Real.0
        // The zero cannot sit at an endpoint, since the signs there are
        // strict.
        if c = a {
            f(a) = f(c)
            f(a) = Real.0
            f(a) < Real.0
            false
        }
        a < c
        if c = b {
            f(b) = f(c)
            f(b) = Real.0
            Real.0 < f(b)
            false
        }
        c < b
        exists(w: Real) { a < w and w < b and f(w) = Real.0 }
    }
}
