// Rolle's theorem (Michel Rolle, 1691; published in his treatise on
// algebra, where it was stated for polynomials): a function continuous on
// a closed interval [a, b], differentiable on the open interval (a, b),
// and taking equal values at the two endpoints must have a horizontal
// tangent somewhere strictly inside: some interior point c satisfies
// f'(c) = 0.  Geometrically, a smooth curve whose endpoints are at the
// same height must be horizontal at some interior point — it has to turn
// around somewhere between the two endpoints.
//
// The theorem is the engine of differential calculus.  The mean value
// theorem (see theorem_mean_value.ac in this directory) follows by
// applying Rolle to the secant-remainder function
//     g(x) = f(x) - s (x - a),   s = (f(b) - f(a)) / (b - a),
// which vanishes at both endpoints and whose derivative is f'(x) - s;
// and from the mean value theorem one derives the monotonicity criteria,
// the constancy of functions with zero derivative, Taylor's theorem,
// l'Hôpital's rule, and the fundamental theorem of calculus.  Cauchy
// (1823) generalized the theorem from polynomials to arbitrary
// differentiable functions.
//
// The formal statement below uses the library's calculus vocabulary
// (`real/mean_value.ac`): `continuous_on_closed(f, a, b)` says that f is
// continuous at every point of the closed interval [a, b],
// `differentiable_on_open(f, a, b)` that f is differentiable at every
// point of the open interval (a, b), and `has_derivative_at(f, c, d)`
// that d is the derivative of f at c (`real/derivative_basic.ac`).
//
// Proof sketch (the classical argument, as in the library's own proof of
// `rolle_theorem` in `real/mean_value.ac`): by the extreme value theorem
// a continuous function on [a, b] attains a maximum and a minimum.  If
// the maximum or the minimum is attained at an interior point c, then
// Fermat's theorem on interior extrema (a differentiable function has
// zero derivative at an interior extremum) gives f'(c) = 0.  If both the
// maximum and the minimum are attained only at the endpoints, the
// equality f(a) = f(b) forces f to be constant on [a, b], in which case
// the derivative is zero at every interior point.
//
// The wrapper below restates the library theorem `rolle_theorem` of
// `real/mean_value.ac`, which is exposed through the public `real`
// interface together with the predicates `continuous_on_closed` and
// `differentiable_on_open`; the library proof is cited directly.

from real import Real, continuous_on_closed, differentiable_on_open, has_derivative_at,
    rolle_theorem

numerals Real

/// Rolle's theorem: a continuous function on a closed interval, equal at
/// the endpoints and differentiable in the interior, has a stationary
/// interior point.
theorem theorems1000_rolle(f: Real -> Real, a: Real, b: Real) {
    continuous_on_closed(f, a, b) and a < b and differentiable_on_open(f, a, b) and f(a) = f(b)
    implies exists(c: Real) {
        a < c and c < b and has_derivative_at(f, c, Real.0)
    }
} by {
    if continuous_on_closed(f, a, b) and a < b and differentiable_on_open(f, a, b) and f(a) = f(b) {
        rolle_theorem(f, a, b)
        exists(c: Real) { a < c and c < b and has_derivative_at(f, c, Real.0) }
    }
}
