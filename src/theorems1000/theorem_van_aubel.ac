from pair import Pair
from real import Real
from top100 import r2_add, r2_neg, r2_sub, r2_dot, r2_norm_sq
from theorems1000.r2_helpers import r2_rot90, r2_rot90_sub, r2_rot90_norm_sq,
    r2_dot_rot90_self, r2_dot_rot90_comm, r2_dot_rot90_swap, r2_double_sub_distrib,
    r2_sub_add_pair_distrib, r2_add_comm, r2_add_assoc, r2_sub_reverse_neg,
    r2_double_eq_double, real_double_eq_double, r2_dot_comm, r2_dot_add_left, r2_dot_add_right,
    r2_dot_sub_left, r2_dot_sub_right, r2_dot_sub_add, r2_norm_sq_add_expansion,
    r2_norm_sq_sub_expansion, r2_norm_sq_double, r2_add_zero_left, real_double_eq_add_self,
    r2_sub_eq_add_neg, r2_sub_sub_sub_rearrange, r2_sub_sub_sum_rearrange

// Van Aubel's theorem: given an arbitrary quadrilateral `abcd`, erect
// squares on its four sides (all with the same orientation), and let `m1`,
// `m2`, `m3`, `m4` be the centers of the squares on `ab`, `bc`, `cd`, `da`
// respectively.  Then the segments `m1 m3` and `m2 m4` joining the centers
// of opposite squares are equal in length and perpendicular:
//     |m1 - m3|^2 = |m2 - m4|^2   and   (m1 - m3) . (m2 - m4) = 0.
//
// The center of the square on `ab` is `m1` with `2 m1 = a + b + R90(b - a)`
// and cyclically, where `R90` is the quarter turn.  Writing `u = a - c` and
// `v = b - d` for the diagonal displacements, the doubled opposite-center
// displacements are
//     2 (m1 - m3) = (u + v) + R90(v - u),
//     2 (m2 - m4) = (v - u) - R90(u + v),
// and the two squared norms agree because `(u + v) . R90(v - u)` is the
// negation of `(v - u) . R90(u + v)` (the quarter turn is skew with respect
// to the dot product); the dot product vanishes because
// `(u + v) . R90(u + v) = 0`, `R90(w) . w = 0`, and
// `R90(u + v) . R90(v - u) = (u + v) . (v - u) = |v|^2 - |u|^2`.

/// The doubled displacement between the centers of opposite squares on the
/// sides `ab` and `cd`:
/// `2 (m1 - m3) = (a - c) + (b - d) + R90((b - d) - (a - c))`.
theorem van_aubel_m1m3(
    a: Pair[Real, Real], b: Pair[Real, Real], c: Pair[Real, Real], d: Pair[Real, Real],
    m1: Pair[Real, Real], m3: Pair[Real, Real]
) {
    r2_add(m1, m1) = r2_add(r2_add(a, b), r2_rot90(r2_sub(b, a))) and
    r2_add(m3, m3) = r2_add(r2_add(c, d), r2_rot90(r2_sub(d, c)))
    implies
    r2_add(r2_sub(m1, m3), r2_sub(m1, m3)) =
        r2_add(
            r2_add(r2_sub(a, c), r2_sub(b, d)),
            r2_rot90(r2_sub(r2_sub(b, d), r2_sub(a, c))))
} by {
    if r2_add(m1, m1) = r2_add(r2_add(a, b), r2_rot90(r2_sub(b, a))) and
        r2_add(m3, m3) = r2_add(r2_add(c, d), r2_rot90(r2_sub(d, c))) {
        r2_add(m1, m1) = r2_add(r2_add(a, b), r2_rot90(r2_sub(b, a)))
        r2_add(m3, m3) = r2_add(r2_add(c, d), r2_rot90(r2_sub(d, c)))
        r2_double_sub_distrib(m1, m3)
        r2_add(r2_sub(m1, m3), r2_sub(m1, m3)) = r2_sub(r2_add(m1, m1), r2_add(m3, m3))
        r2_sub(r2_add(m1, m1), r2_add(m3, m3)) =
            r2_sub(
                r2_add(r2_add(a, b), r2_rot90(r2_sub(b, a))),
                r2_add(r2_add(c, d), r2_rot90(r2_sub(d, c))))
        r2_sub_add_pair_distrib(
            r2_add(a, b), r2_add(c, d), r2_rot90(r2_sub(b, a)), r2_rot90(r2_sub(d, c)))
        r2_sub(
            r2_add(r2_add(a, b), r2_rot90(r2_sub(b, a))),
            r2_add(r2_add(c, d), r2_rot90(r2_sub(d, c)))) =
            r2_add(
                r2_sub(r2_add(a, b), r2_add(c, d)),
                r2_sub(r2_rot90(r2_sub(b, a)), r2_rot90(r2_sub(d, c))))
        r2_sub_add_pair_distrib(a, c, b, d)
        r2_sub(r2_add(a, b), r2_add(c, d)) = r2_add(r2_sub(a, c), r2_sub(b, d))
        r2_rot90_sub(r2_sub(b, a), r2_sub(d, c))
        r2_rot90(r2_sub(r2_sub(b, a), r2_sub(d, c))) =
            r2_sub(r2_rot90(r2_sub(b, a)), r2_rot90(r2_sub(d, c)))
        r2_sub(r2_rot90(r2_sub(b, a)), r2_rot90(r2_sub(d, c))) =
            r2_rot90(r2_sub(r2_sub(b, a), r2_sub(d, c)))
        r2_sub_sub_sub_rearrange(a, b, c, d)
        r2_sub(r2_sub(b, a), r2_sub(d, c)) = r2_sub(r2_sub(b, d), r2_sub(a, c))
        r2_sub(r2_rot90(r2_sub(b, a)), r2_rot90(r2_sub(d, c))) =
            r2_rot90(r2_sub(r2_sub(b, d), r2_sub(a, c)))
        r2_sub(r2_add(m1, m1), r2_add(m3, m3)) =
            r2_add(
                r2_add(r2_sub(a, c), r2_sub(b, d)),
                r2_rot90(r2_sub(r2_sub(b, d), r2_sub(a, c))))
        r2_add(r2_sub(m1, m3), r2_sub(m1, m3)) =
            r2_add(
                r2_add(r2_sub(a, c), r2_sub(b, d)),
                r2_rot90(r2_sub(r2_sub(b, d), r2_sub(a, c))))
    }
}

/// The doubled displacement between the centers of opposite squares on the
/// sides `bc` and `da`:
/// `2 (m2 - m4) = ((b - d) - (a - c)) - R90((a - c) + (b - d))`.
theorem van_aubel_m2m4(
    a: Pair[Real, Real], b: Pair[Real, Real], c: Pair[Real, Real], d: Pair[Real, Real],
    m2: Pair[Real, Real], m4: Pair[Real, Real]
) {
    r2_add(m2, m2) = r2_add(r2_add(b, c), r2_rot90(r2_sub(c, b))) and
    r2_add(m4, m4) = r2_add(r2_add(d, a), r2_rot90(r2_sub(a, d)))
    implies
    r2_add(r2_sub(m2, m4), r2_sub(m2, m4)) =
        r2_sub(
            r2_sub(r2_sub(b, d), r2_sub(a, c)),
            r2_rot90(r2_add(r2_sub(a, c), r2_sub(b, d))))
} by {
    if r2_add(m2, m2) = r2_add(r2_add(b, c), r2_rot90(r2_sub(c, b))) and
        r2_add(m4, m4) = r2_add(r2_add(d, a), r2_rot90(r2_sub(a, d))) {
        r2_add(m2, m2) = r2_add(r2_add(b, c), r2_rot90(r2_sub(c, b)))
        r2_add(m4, m4) = r2_add(r2_add(d, a), r2_rot90(r2_sub(a, d)))
        r2_double_sub_distrib(m2, m4)
        r2_add(r2_sub(m2, m4), r2_sub(m2, m4)) = r2_sub(r2_add(m2, m2), r2_add(m4, m4))
        r2_sub(r2_add(m2, m2), r2_add(m4, m4)) =
            r2_sub(
                r2_add(r2_add(b, c), r2_rot90(r2_sub(c, b))),
                r2_add(r2_add(d, a), r2_rot90(r2_sub(a, d))))
        r2_sub_add_pair_distrib(
            r2_add(b, c), r2_add(d, a), r2_rot90(r2_sub(c, b)), r2_rot90(r2_sub(a, d)))
        r2_sub(
            r2_add(r2_add(b, c), r2_rot90(r2_sub(c, b))),
            r2_add(r2_add(d, a), r2_rot90(r2_sub(a, d)))) =
            r2_add(
                r2_sub(r2_add(b, c), r2_add(d, a)),
                r2_sub(r2_rot90(r2_sub(c, b)), r2_rot90(r2_sub(a, d))))
        // (b + c) - (d + a) = (b - d) - (a - c)
        r2_sub_add_pair_distrib(b, d, c, a)
        r2_sub(r2_add(b, c), r2_add(d, a)) = r2_add(r2_sub(b, d), r2_sub(c, a))
        r2_sub_reverse_neg(a, c)
        r2_sub(a, c) = r2_neg(r2_sub(c, a))
        r2_neg(r2_sub(c, a)) = r2_sub(a, c)
        r2_sub(c, a) = r2_neg(r2_sub(a, c))
        r2_add(r2_sub(b, d), r2_sub(c, a)) = r2_add(r2_sub(b, d), r2_neg(r2_sub(a, c)))
        r2_sub_eq_add_neg(r2_sub(b, d), r2_sub(a, c))
        r2_sub(r2_sub(b, d), r2_sub(a, c)) = r2_add(r2_sub(b, d), r2_neg(r2_sub(a, c)))
        r2_add(r2_sub(b, d), r2_sub(c, a)) = r2_sub(r2_sub(b, d), r2_sub(a, c))
        r2_sub(r2_add(b, c), r2_add(d, a)) = r2_sub(r2_sub(b, d), r2_sub(a, c))
        // R90(c - b) - R90(a - d) = -R90((a - c) + (b - d))
        r2_rot90_sub(r2_sub(c, b), r2_sub(a, d))
        r2_rot90(r2_sub(r2_sub(c, b), r2_sub(a, d))) =
            r2_sub(r2_rot90(r2_sub(c, b)), r2_rot90(r2_sub(a, d)))
        r2_sub(r2_rot90(r2_sub(c, b)), r2_rot90(r2_sub(a, d))) =
            r2_rot90(r2_sub(r2_sub(c, b), r2_sub(a, d)))
        r2_sub_sub_sum_rearrange(a, b, c, d)
        r2_sub(r2_sub(c, b), r2_sub(a, d)) = r2_sub(r2_add(c, d), r2_add(a, b))
        r2_sub(r2_rot90(r2_sub(c, b)), r2_rot90(r2_sub(a, d))) =
            r2_rot90(r2_sub(r2_add(c, d), r2_add(a, b)))
        r2_sub(r2_add(c, d), r2_add(a, b)) = r2_neg(r2_add(r2_sub(a, c), r2_sub(b, d)))
        r2_rot90(r2_sub(r2_add(c, d), r2_add(a, b))) =
            r2_rot90(r2_neg(r2_add(r2_sub(a, c), r2_sub(b, d))))
        r2_rot90(r2_neg(r2_add(r2_sub(a, c), r2_sub(b, d)))) =
            r2_neg(r2_rot90(r2_add(r2_sub(a, c), r2_sub(b, d))))
        r2_sub(r2_rot90(r2_sub(c, b)), r2_rot90(r2_sub(a, d))) =
            r2_neg(r2_rot90(r2_add(r2_sub(a, c), r2_sub(b, d))))
        r2_sub(r2_add(m2, m2), r2_add(m4, m4)) =
            r2_add(
                r2_sub(r2_sub(b, d), r2_sub(a, c)),
                r2_neg(r2_rot90(r2_add(r2_sub(a, c), r2_sub(b, d)))))
        r2_sub_eq_add_neg(r2_sub(r2_sub(b, d), r2_sub(a, c)), r2_rot90(r2_add(r2_sub(a, c), r2_sub(b, d))))
        r2_sub(r2_sub(r2_sub(b, d), r2_sub(a, c)), r2_rot90(r2_add(r2_sub(a, c), r2_sub(b, d)))) =
            r2_add(
                r2_sub(r2_sub(b, d), r2_sub(a, c)),
                r2_neg(r2_rot90(r2_add(r2_sub(a, c), r2_sub(b, d)))))
        r2_sub(r2_add(m2, m2), r2_add(m4, m4)) =
            r2_sub(
                r2_sub(r2_sub(b, d), r2_sub(a, c)),
                r2_rot90(r2_add(r2_sub(a, c), r2_sub(b, d))))
        r2_add(r2_sub(m2, m4), r2_sub(m2, m4)) =
            r2_sub(
                r2_sub(r2_sub(b, d), r2_sub(a, c)),
                r2_rot90(r2_add(r2_sub(a, c), r2_sub(b, d))))
    }
}

/// The squared norm of `u + v + R90(v - u)` expands completely.
theorem van_aubel_norm_rot90_sum(u: Pair[Real, Real], v: Pair[Real, Real]) {
    r2_norm_sq(r2_add(r2_add(u, v), r2_rot90(r2_sub(v, u)))) =
        r2_norm_sq(r2_add(u, v)) + r2_norm_sq(r2_sub(v, u)) +
        r2_dot(r2_add(u, v), r2_rot90(r2_sub(v, u))) +
        r2_dot(r2_add(u, v), r2_rot90(r2_sub(v, u)))
} by {
    r2_norm_sq_add_expansion(r2_add(u, v), r2_rot90(r2_sub(v, u)))
    r2_norm_sq(r2_add(r2_add(u, v), r2_rot90(r2_sub(v, u)))) =
        r2_norm_sq(r2_add(u, v)) + r2_norm_sq(r2_rot90(r2_sub(v, u))) +
        r2_dot(r2_add(u, v), r2_rot90(r2_sub(v, u))) +
        r2_dot(r2_add(u, v), r2_rot90(r2_sub(v, u)))
    r2_rot90_norm_sq(r2_sub(v, u))
    r2_norm_sq(r2_rot90(r2_sub(v, u))) = r2_norm_sq(r2_sub(v, u))
    r2_norm_sq(r2_add(r2_add(u, v), r2_rot90(r2_sub(v, u)))) =
        r2_norm_sq(r2_add(u, v)) + r2_norm_sq(r2_sub(v, u)) +
        r2_dot(r2_add(u, v), r2_rot90(r2_sub(v, u))) +
        r2_dot(r2_add(u, v), r2_rot90(r2_sub(v, u)))
}

/// The squared norm of `(v - u) - R90(u + v)` expands completely.
theorem van_aubel_norm_rot90_sub(u: Pair[Real, Real], v: Pair[Real, Real]) {
    r2_norm_sq(r2_sub(r2_sub(v, u), r2_rot90(r2_add(u, v)))) =
        r2_norm_sq(r2_sub(v, u)) + r2_norm_sq(r2_add(u, v)) -
        r2_dot(r2_sub(v, u), r2_rot90(r2_add(u, v))) -
        r2_dot(r2_sub(v, u), r2_rot90(r2_add(u, v)))
} by {
    r2_norm_sq_sub_expansion(r2_sub(v, u), r2_rot90(r2_add(u, v)))
    r2_norm_sq(r2_sub(r2_sub(v, u), r2_rot90(r2_add(u, v)))) =
        r2_norm_sq(r2_sub(v, u)) + r2_norm_sq(r2_rot90(r2_add(u, v))) -
        r2_dot(r2_sub(v, u), r2_rot90(r2_add(u, v))) -
        r2_dot(r2_sub(v, u), r2_rot90(r2_add(u, v)))
    r2_rot90_norm_sq(r2_add(u, v))
    r2_norm_sq(r2_rot90(r2_add(u, v))) = r2_norm_sq(r2_add(u, v))
    r2_norm_sq(r2_sub(r2_sub(v, u), r2_rot90(r2_add(u, v)))) =
        r2_norm_sq(r2_sub(v, u)) + r2_norm_sq(r2_add(u, v)) -
        r2_dot(r2_sub(v, u), r2_rot90(r2_add(u, v))) -
        r2_dot(r2_sub(v, u), r2_rot90(r2_add(u, v)))
}

/// A sum with a doubled term may be rearranged:
/// `p + q + a + a = q + p + a + a`.
theorem van_aubel_add_rearrange(p: Real, q: Real, a: Real) {
    p + q + a + a = q + p + a + a
} by {
    p + q + a + a = (p + q) + (a + a)
    (p + q) + (a + a) = (q + p) + (a + a)
    (q + p) + (a + a) = q + p + a + a
}

/// A doubled subtraction may be written with negations:
/// `p + q - x - x = p + q + -x + -x`.
theorem van_aubel_neg_neg_form(p: Real, q: Real, x: Real) {
    p + q - x - x = p + q + -x + -x
} by {
    p + q - x - x = (p + q) - x - x
    (p + q) - x = (p + q) + -x
    (p + q) - x - x = (p + q) + -x - x
    (p + q) + -x - x = (p + q) + -x + -x
    (p + q) + -x + -x = p + q + -x + -x
}

/// The two doubled opposite-center displacements have equal squared norms.
theorem van_aubel_norm_equality(u: Pair[Real, Real], v: Pair[Real, Real]) {
    r2_norm_sq(r2_add(r2_add(u, v), r2_rot90(r2_sub(v, u)))) =
        r2_norm_sq(r2_sub(r2_sub(v, u), r2_rot90(r2_add(u, v))))
} by {
    van_aubel_norm_rot90_sum(u, v)
    r2_norm_sq(r2_add(r2_add(u, v), r2_rot90(r2_sub(v, u)))) =
        r2_norm_sq(r2_add(u, v)) + r2_norm_sq(r2_sub(v, u)) +
        r2_dot(r2_add(u, v), r2_rot90(r2_sub(v, u))) +
        r2_dot(r2_add(u, v), r2_rot90(r2_sub(v, u)))
    van_aubel_norm_rot90_sub(u, v)
    r2_norm_sq(r2_sub(r2_sub(v, u), r2_rot90(r2_add(u, v)))) =
        r2_norm_sq(r2_sub(v, u)) + r2_norm_sq(r2_add(u, v)) -
        r2_dot(r2_sub(v, u), r2_rot90(r2_add(u, v))) -
        r2_dot(r2_sub(v, u), r2_rot90(r2_add(u, v)))
    r2_dot_rot90_swap(r2_add(u, v), r2_sub(v, u))
    r2_dot(r2_add(u, v), r2_rot90(r2_sub(v, u))) = -r2_dot(r2_sub(v, u), r2_rot90(r2_add(u, v)))
    r2_norm_sq(r2_add(u, v)) + r2_norm_sq(r2_sub(v, u)) +
        r2_dot(r2_add(u, v), r2_rot90(r2_sub(v, u))) +
        r2_dot(r2_add(u, v), r2_rot90(r2_sub(v, u))) =
        r2_norm_sq(r2_add(u, v)) + r2_norm_sq(r2_sub(v, u)) +
        -r2_dot(r2_sub(v, u), r2_rot90(r2_add(u, v))) +
        -r2_dot(r2_sub(v, u), r2_rot90(r2_add(u, v)))
    r2_norm_sq(r2_add(r2_add(u, v), r2_rot90(r2_sub(v, u)))) =
        r2_norm_sq(r2_add(u, v)) + r2_norm_sq(r2_sub(v, u)) +
        -r2_dot(r2_sub(v, u), r2_rot90(r2_add(u, v))) +
        -r2_dot(r2_sub(v, u), r2_rot90(r2_add(u, v)))
    van_aubel_add_rearrange(
        r2_norm_sq(r2_add(u, v)), r2_norm_sq(r2_sub(v, u)),
        -r2_dot(r2_sub(v, u), r2_rot90(r2_add(u, v))))
    r2_norm_sq(r2_add(u, v)) + r2_norm_sq(r2_sub(v, u)) +
        -r2_dot(r2_sub(v, u), r2_rot90(r2_add(u, v))) +
        -r2_dot(r2_sub(v, u), r2_rot90(r2_add(u, v))) =
        r2_norm_sq(r2_sub(v, u)) + r2_norm_sq(r2_add(u, v)) +
        -r2_dot(r2_sub(v, u), r2_rot90(r2_add(u, v))) +
        -r2_dot(r2_sub(v, u), r2_rot90(r2_add(u, v)))
    r2_norm_sq(r2_add(r2_add(u, v), r2_rot90(r2_sub(v, u)))) =
        r2_norm_sq(r2_sub(v, u)) + r2_norm_sq(r2_add(u, v)) +
        -r2_dot(r2_sub(v, u), r2_rot90(r2_add(u, v))) +
        -r2_dot(r2_sub(v, u), r2_rot90(r2_add(u, v)))
    van_aubel_neg_neg_form(
        r2_norm_sq(r2_sub(v, u)), r2_norm_sq(r2_add(u, v)),
        r2_dot(r2_sub(v, u), r2_rot90(r2_add(u, v))))
    r2_norm_sq(r2_sub(v, u)) + r2_norm_sq(r2_add(u, v)) -
        r2_dot(r2_sub(v, u), r2_rot90(r2_add(u, v))) -
        r2_dot(r2_sub(v, u), r2_rot90(r2_add(u, v))) =
        r2_norm_sq(r2_sub(v, u)) + r2_norm_sq(r2_add(u, v)) +
        -r2_dot(r2_sub(v, u), r2_rot90(r2_add(u, v))) +
        -r2_dot(r2_sub(v, u), r2_rot90(r2_add(u, v)))
    r2_norm_sq(r2_sub(r2_sub(v, u), r2_rot90(r2_add(u, v)))) =
        r2_norm_sq(r2_sub(v, u)) + r2_norm_sq(r2_add(u, v)) +
        -r2_dot(r2_sub(v, u), r2_rot90(r2_add(u, v))) +
        -r2_dot(r2_sub(v, u), r2_rot90(r2_add(u, v)))
    r2_norm_sq(r2_add(r2_add(u, v), r2_rot90(r2_sub(v, u)))) =
        r2_norm_sq(r2_sub(r2_sub(v, u), r2_rot90(r2_add(u, v))))
}

/// A quadrupled real equality cancels to the unquadrupled equality.
theorem van_aubel_norm_cancel(x: Real, y: Real) {
    (Real.1 + Real.1) * ((Real.1 + Real.1) * x) =
        (Real.1 + Real.1) * ((Real.1 + Real.1) * y)
    implies x = y
} by {
    if (Real.1 + Real.1) * ((Real.1 + Real.1) * x) =
        (Real.1 + Real.1) * ((Real.1 + Real.1) * y) {
        real_double_eq_add_self(x)
        real_double_eq_add_self(y)
        real_double_eq_add_self((Real.1 + Real.1) * x)
        real_double_eq_add_self((Real.1 + Real.1) * y)
        (Real.1 + Real.1) * ((Real.1 + Real.1) * x) = ((Real.1 + Real.1) * x) + ((Real.1 + Real.1) * x)
        (Real.1 + Real.1) * ((Real.1 + Real.1) * y) = ((Real.1 + Real.1) * y) + ((Real.1 + Real.1) * y)
        ((Real.1 + Real.1) * x) + ((Real.1 + Real.1) * x) =
            ((Real.1 + Real.1) * y) + ((Real.1 + Real.1) * y)
        real_double_eq_double((Real.1 + Real.1) * x, (Real.1 + Real.1) * y)
        (Real.1 + Real.1) * x = (Real.1 + Real.1) * y
        (Real.1 + Real.1) * x = x + x
        (Real.1 + Real.1) * y = y + y
        x + x = y + y
        real_double_eq_double(x, y)
        x = y
    }
}

/// The dot product of the two doubled opposite-center displacements is zero.
theorem van_aubel_dot_zero(u: Pair[Real, Real], v: Pair[Real, Real]) {
    r2_dot(
        r2_add(r2_add(u, v), r2_rot90(r2_sub(v, u))),
        r2_sub(r2_sub(v, u), r2_rot90(r2_add(u, v)))) = Real.0
} by {
    r2_dot_add_left(r2_add(u, v), r2_rot90(r2_sub(v, u)), r2_sub(r2_sub(v, u), r2_rot90(r2_add(u, v))))
    r2_dot(
        r2_add(r2_add(u, v), r2_rot90(r2_sub(v, u))),
        r2_sub(r2_sub(v, u), r2_rot90(r2_add(u, v)))) =
        r2_dot(r2_add(u, v), r2_sub(r2_sub(v, u), r2_rot90(r2_add(u, v)))) +
        r2_dot(r2_rot90(r2_sub(v, u)), r2_sub(r2_sub(v, u), r2_rot90(r2_add(u, v))))
    r2_dot_sub_right(r2_add(u, v), r2_sub(v, u), r2_rot90(r2_add(u, v)))
    r2_dot(r2_add(u, v), r2_sub(r2_sub(v, u), r2_rot90(r2_add(u, v)))) =
        r2_dot(r2_add(u, v), r2_sub(v, u)) - r2_dot(r2_add(u, v), r2_rot90(r2_add(u, v)))
    r2_dot_sub_right(r2_rot90(r2_sub(v, u)), r2_sub(v, u), r2_rot90(r2_add(u, v)))
    r2_dot(r2_rot90(r2_sub(v, u)), r2_sub(r2_sub(v, u), r2_rot90(r2_add(u, v)))) =
        r2_dot(r2_rot90(r2_sub(v, u)), r2_sub(v, u)) -
        r2_dot(r2_rot90(r2_sub(v, u)), r2_rot90(r2_add(u, v)))
    // (u + v) . R90(u + v) = 0 and R90(v - u) . (v - u) = 0
    r2_dot_rot90_self(r2_add(u, v))
    r2_dot(r2_add(u, v), r2_rot90(r2_add(u, v))) = Real.0
    r2_dot(r2_add(u, v), r2_sub(r2_sub(v, u), r2_rot90(r2_add(u, v)))) =
        r2_dot(r2_add(u, v), r2_sub(v, u)) - Real.0
    r2_dot_comm(r2_rot90(r2_sub(v, u)), r2_sub(v, u))
    r2_dot(r2_rot90(r2_sub(v, u)), r2_sub(v, u)) = r2_dot(r2_sub(v, u), r2_rot90(r2_sub(v, u)))
    r2_dot_rot90_self(r2_sub(v, u))
    r2_dot(r2_sub(v, u), r2_rot90(r2_sub(v, u))) = Real.0
    r2_dot(r2_rot90(r2_sub(v, u)), r2_sub(v, u)) = Real.0
    r2_dot(r2_rot90(r2_sub(v, u)), r2_sub(r2_sub(v, u), r2_rot90(r2_add(u, v)))) =
        Real.0 - r2_dot(r2_rot90(r2_sub(v, u)), r2_rot90(r2_add(u, v)))
    // the assembled expansion
    r2_dot(
        r2_add(r2_add(u, v), r2_rot90(r2_sub(v, u))),
        r2_sub(r2_sub(v, u), r2_rot90(r2_add(u, v)))) =
        (r2_dot(r2_add(u, v), r2_sub(v, u)) - Real.0) +
        (Real.0 - r2_dot(r2_rot90(r2_sub(v, u)), r2_rot90(r2_add(u, v))))
    (r2_dot(r2_add(u, v), r2_sub(v, u)) - Real.0) +
        (Real.0 - r2_dot(r2_rot90(r2_sub(v, u)), r2_rot90(r2_add(u, v)))) =
        r2_dot(r2_add(u, v), r2_sub(v, u)) -
        r2_dot(r2_rot90(r2_sub(v, u)), r2_rot90(r2_add(u, v)))
    r2_dot(
        r2_add(r2_add(u, v), r2_rot90(r2_sub(v, u))),
        r2_sub(r2_sub(v, u), r2_rot90(r2_add(u, v)))) =
        r2_dot(r2_add(u, v), r2_sub(v, u)) -
        r2_dot(r2_rot90(r2_sub(v, u)), r2_rot90(r2_add(u, v)))
    // both terms are (v - u) . (u + v) = |v|^2 - |u|^2
    r2_dot_rot90_comm(r2_sub(v, u), r2_add(u, v))
    r2_dot(r2_rot90(r2_sub(v, u)), r2_rot90(r2_add(u, v))) =
        r2_dot(r2_sub(v, u), r2_add(u, v))
    r2_dot_comm(r2_add(u, v), r2_sub(v, u))
    r2_dot(r2_add(u, v), r2_sub(v, u)) = r2_dot(r2_sub(v, u), r2_add(u, v))
    r2_dot(
        r2_add(r2_add(u, v), r2_rot90(r2_sub(v, u))),
        r2_sub(r2_sub(v, u), r2_rot90(r2_add(u, v)))) =
        r2_dot(r2_sub(v, u), r2_add(u, v)) - r2_dot(r2_sub(v, u), r2_add(u, v))
    r2_dot(r2_sub(v, u), r2_add(u, v)) - r2_dot(r2_sub(v, u), r2_add(u, v)) = Real.0
    r2_dot(
        r2_add(r2_add(u, v), r2_rot90(r2_sub(v, u))),
        r2_sub(r2_sub(v, u), r2_rot90(r2_add(u, v)))) = Real.0
}

/// The dot product of doubled vectors is quadruple the dot product.
theorem r2_dot_double_double(x: Pair[Real, Real], y: Pair[Real, Real]) {
    r2_dot(r2_add(x, x), r2_add(y, y)) =
        (Real.1 + Real.1) * ((Real.1 + Real.1) * r2_dot(x, y))
} by {
    r2_dot_add_left(x, x, r2_add(y, y))
    r2_dot(r2_add(x, x), r2_add(y, y)) = r2_dot(x, r2_add(y, y)) + r2_dot(x, r2_add(y, y))
    r2_dot_add_right(x, y, y)
    r2_dot(x, r2_add(y, y)) = r2_dot(x, y) + r2_dot(x, y)
    real_double_eq_add_self(r2_dot(x, y))
    (Real.1 + Real.1) * r2_dot(x, y) = r2_dot(x, y) + r2_dot(x, y)
    r2_dot(x, r2_add(y, y)) = (Real.1 + Real.1) * r2_dot(x, y)
    r2_dot(r2_add(x, x), r2_add(y, y)) =
        (Real.1 + Real.1) * r2_dot(x, y) + (Real.1 + Real.1) * r2_dot(x, y)
    real_double_eq_add_self((Real.1 + Real.1) * r2_dot(x, y))
    (Real.1 + Real.1) * ((Real.1 + Real.1) * r2_dot(x, y)) =
        (Real.1 + Real.1) * r2_dot(x, y) + (Real.1 + Real.1) * r2_dot(x, y)
    r2_dot(r2_add(x, x), r2_add(y, y)) =
        (Real.1 + Real.1) * ((Real.1 + Real.1) * r2_dot(x, y))
}

/// A quadrupled zero dot product cancels.
theorem van_aubel_dot_cancel(d: Real) {
    (Real.1 + Real.1) * ((Real.1 + Real.1) * d) = Real.0 implies d = Real.0
} by {
    if (Real.1 + Real.1) * ((Real.1 + Real.1) * d) = Real.0 {
        real_double_eq_add_self(d)
        real_double_eq_add_self((Real.1 + Real.1) * d)
        (Real.1 + Real.1) * d = d + d
        (Real.1 + Real.1) * ((Real.1 + Real.1) * d) = ((Real.1 + Real.1) * d) + ((Real.1 + Real.1) * d)
        ((Real.1 + Real.1) * d) + ((Real.1 + Real.1) * d) = Real.0
        ((Real.1 + Real.1) * d) + ((Real.1 + Real.1) * d) = Real.0 + Real.0
        real_double_eq_double((Real.1 + Real.1) * d, Real.0)
        (Real.1 + Real.1) * d = Real.0
        d + d = Real.0
        d + d = Real.0 + Real.0
        real_double_eq_double(d, Real.0)
        d = Real.0
    }
}

/// Van Aubel's theorem.
///
/// Squares are erected on the four sides of the quadrilateral `abcd`, with
/// `m1`, `m2`, `m3`, `m4` the centers of the squares on `ab`, `bc`, `cd`,
/// `da`.  The segments joining the centers of opposite squares are equal in
/// length and perpendicular:
/// `|m1 - m3|^2 = |m2 - m4|^2` and `(m1 - m3) . (m2 - m4) = 0`.
theorem theorems1000_van_aubel(
    a: Pair[Real, Real], b: Pair[Real, Real], c: Pair[Real, Real], d: Pair[Real, Real],
    m1: Pair[Real, Real], m2: Pair[Real, Real], m3: Pair[Real, Real], m4: Pair[Real, Real]
) {
    r2_add(m1, m1) = r2_add(r2_add(a, b), r2_rot90(r2_sub(b, a))) and
    r2_add(m2, m2) = r2_add(r2_add(b, c), r2_rot90(r2_sub(c, b))) and
    r2_add(m3, m3) = r2_add(r2_add(c, d), r2_rot90(r2_sub(d, c))) and
    r2_add(m4, m4) = r2_add(r2_add(d, a), r2_rot90(r2_sub(a, d)))
    implies
    r2_norm_sq(r2_sub(m1, m3)) = r2_norm_sq(r2_sub(m2, m4)) and
    r2_dot(r2_sub(m1, m3), r2_sub(m2, m4)) = Real.0
} by {
    if r2_add(m1, m1) = r2_add(r2_add(a, b), r2_rot90(r2_sub(b, a))) and
        r2_add(m2, m2) = r2_add(r2_add(b, c), r2_rot90(r2_sub(c, b))) and
        r2_add(m3, m3) = r2_add(r2_add(c, d), r2_rot90(r2_sub(d, c))) and
        r2_add(m4, m4) = r2_add(r2_add(d, a), r2_rot90(r2_sub(a, d))) {
        r2_add(m1, m1) = r2_add(r2_add(a, b), r2_rot90(r2_sub(b, a)))
        r2_add(m2, m2) = r2_add(r2_add(b, c), r2_rot90(r2_sub(c, b)))
        r2_add(m3, m3) = r2_add(r2_add(c, d), r2_rot90(r2_sub(d, c)))
        r2_add(m4, m4) = r2_add(r2_add(d, a), r2_rot90(r2_sub(a, d)))
        // the doubled opposite-center displacements
        van_aubel_m1m3(a, b, c, d, m1, m3)
        r2_add(r2_sub(m1, m3), r2_sub(m1, m3)) =
            r2_add(
                r2_add(r2_sub(a, c), r2_sub(b, d)),
                r2_rot90(r2_sub(r2_sub(b, d), r2_sub(a, c))))
        van_aubel_m2m4(a, b, c, d, m2, m4)
        r2_add(r2_sub(m2, m4), r2_sub(m2, m4)) =
            r2_sub(
                r2_sub(r2_sub(b, d), r2_sub(a, c)),
                r2_rot90(r2_add(r2_sub(a, c), r2_sub(b, d))))
        // equal squared lengths
        van_aubel_norm_equality(r2_sub(a, c), r2_sub(b, d))
        r2_norm_sq(r2_add(r2_add(r2_sub(a, c), r2_sub(b, d)), r2_rot90(r2_sub(r2_sub(b, d), r2_sub(a, c))))) =
            r2_norm_sq(r2_sub(r2_sub(r2_sub(b, d), r2_sub(a, c)), r2_rot90(r2_add(r2_sub(a, c), r2_sub(b, d)))))
        r2_norm_sq_double(r2_sub(m1, m3))
        r2_norm_sq(r2_add(r2_sub(m1, m3), r2_sub(m1, m3))) =
            (Real.1 + Real.1) * ((Real.1 + Real.1) * r2_norm_sq(r2_sub(m1, m3)))
        r2_norm_sq_double(r2_sub(m2, m4))
        r2_norm_sq(r2_add(r2_sub(m2, m4), r2_sub(m2, m4))) =
            (Real.1 + Real.1) * ((Real.1 + Real.1) * r2_norm_sq(r2_sub(m2, m4)))
        r2_norm_sq(r2_add(r2_sub(m1, m3), r2_sub(m1, m3))) =
            r2_norm_sq(r2_add(r2_sub(m2, m4), r2_sub(m2, m4)))
        (Real.1 + Real.1) * ((Real.1 + Real.1) * r2_norm_sq(r2_sub(m1, m3))) =
            (Real.1 + Real.1) * ((Real.1 + Real.1) * r2_norm_sq(r2_sub(m2, m4)))
        van_aubel_norm_cancel(r2_norm_sq(r2_sub(m1, m3)), r2_norm_sq(r2_sub(m2, m4)))
        r2_norm_sq(r2_sub(m1, m3)) = r2_norm_sq(r2_sub(m2, m4))
        // perpendicularity
        van_aubel_dot_zero(r2_sub(a, c), r2_sub(b, d))
        r2_dot(
            r2_add(r2_add(r2_sub(a, c), r2_sub(b, d)), r2_rot90(r2_sub(r2_sub(b, d), r2_sub(a, c)))),
            r2_sub(r2_sub(r2_sub(b, d), r2_sub(a, c)), r2_rot90(r2_add(r2_sub(a, c), r2_sub(b, d))))) = Real.0
        r2_dot(
            r2_add(r2_sub(m1, m3), r2_sub(m1, m3)),
            r2_add(r2_sub(m2, m4), r2_sub(m2, m4))) = Real.0
        r2_dot_double_double(r2_sub(m1, m3), r2_sub(m2, m4))
        r2_dot(r2_add(r2_sub(m1, m3), r2_sub(m1, m3)), r2_add(r2_sub(m2, m4), r2_sub(m2, m4))) =
            (Real.1 + Real.1) * ((Real.1 + Real.1) * r2_dot(r2_sub(m1, m3), r2_sub(m2, m4)))
        (Real.1 + Real.1) * ((Real.1 + Real.1) * r2_dot(r2_sub(m1, m3), r2_sub(m2, m4))) = Real.0
        van_aubel_dot_cancel(r2_dot(r2_sub(m1, m3), r2_sub(m2, m4)))
        r2_dot(r2_sub(m1, m3), r2_sub(m2, m4)) = Real.0
        r2_norm_sq(r2_sub(m1, m3)) = r2_norm_sq(r2_sub(m2, m4)) and
            r2_dot(r2_sub(m1, m3), r2_sub(m2, m4)) = Real.0
    }
}
