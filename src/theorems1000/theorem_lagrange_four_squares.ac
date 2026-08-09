from nat import Nat

numerals Nat

// Lagrange's four-square theorem: every natural number is the sum of four
// squares of natural numbers,
//     n = a^2 + b^2 + c^2 + d^2.
// Joseph-Louis Lagrange proved the theorem in 1770; it answers Waring's
// problem for squares (the case `g(2) = 4` of the Hilbert-Waring theorem,
// theorem_hilbert_waring.ac in this directory), and it is the basis of
// Euler's proof of the four-square theorem for the quaternion norm.
//
// The proof has two ingredients, both of which are recorded here.
//
// (1) Euler's four-square identity: the product of two sums of four
// squares is again a sum of four squares.  With the quaternion product
//     (a + b i + c j + d k)(e + f i + g j + h k)
//         = q1 + q2 i + q3 j + q4 k,
//     q1 = a e - b f - c g - d h,
//     q2 = a f + b e + c h - d g,
//     q3 = a g - b h + c e + d f,
//     q4 = a h + b g - c f + d e,
// the multiplicativity of the squared norm `|q r|^2 = |q|^2 |r|^2` is the
// polynomial identity
//     (a^2 + b^2 + c^2 + d^2)(e^2 + f^2 + g^2 + h^2) = q1^2 + q2^2 + q3^2 + q4^2.
// Expanding the four squares, all twelve pairs of cross terms cancel
// (each pair is four copies of the same monomial, two positive and two
// negative), leaving the sixteen pure terms, exactly the expansion of the
// left-hand side.  The proof is a longer version of the two-dimensional
// Lagrange identity proved in theorem_brahmagupta_fibonacci.ac (or the
// three-dimensional one in theorem_lagrange_identity.ac), using the
// cancellation lemmas of ring_helpers.ac in this directory; each step
// verified during development, but the full chain of monomial
// reorderings does not terminate reliably, so the identity is recorded
// here as a statement.
//
// theorem theorems1000_euler_four_square_identity(
//     a: Real, b: Real, c: Real, d: Real,
//     e: Real, f: Real, g: Real, h: Real
// ) {
//     (a * a + b * b + c * c + d * d) * (e * e + f * f + g * g + h * h) =
//         (a * e - b * f - c * g - d * h) * (a * e - b * f - c * g - d * h) +
//         (a * f + b * e + c * h - d * g) * (a * f + b * e + c * h - d * g) +
//         (a * g - b * h + c * e + d * f) * (a * g - b * h + c * e + d * f) +
//         (a * h + b * g - c * f + d * e) * (a * h + b * g - c * f + d * e)
// }
//
// (2) The descent argument: it suffices to prove the theorem for primes,
// since by Euler's identity the representable numbers are closed under
// multiplication.  For an odd prime `p`, the pigeonhole principle gives
// integers `x, y` with `x^2 + y^2 = m p` for some `m < p`; the
// two-square identity (theorem_brahmagupta_fibonacci.ac) then "descends"
// the representation to a smaller multiple of `p`, and repeating the
// argument reaches `m = 1`.  The base cases 0, 1, 2 are immediate.  The
// library's `number_theory/four_squares.ac` develops the closure machinery
// (`is_sum_four_squares`, multiplication by squares, and the 
// `is_sum_four_squares_*` witnesses); the descent step needs the
// pigeonhole principle for residues modulo a prime, which is not yet
// formalised.
//
// theorem theorems1000_lagrange_four_squares(n: Nat) {
//     exists(a: Nat, b: Nat, c: Nat, d: Nat) {
//         n = a * a + b * b + c * c + d * d
//     }
// }
