from real import Real

// The Jordan curve theorem (Camille Jordan, 1887): every simple closed
// curve in the plane separates the plane into exactly two connected
// regions, one bounded (the interior) and one unbounded (the exterior),
// and the curve is the common boundary of both.  "Simple closed curve"
// means the image of the unit circle under a continuous injective map; the
// conclusion is famously obvious and notoriously hard to prove — Jordan's
// own proof was incomplete, and the first complete proofs (Veblen 1905,
// Schoenflies 1906) came almost twenty years later.  The theorem is the
// topological foundation of complex analysis (the Cauchy integral theorem
// and the argument principle rest on it) and of planar graph theory.
//
// The formal statement below encodes the curve as the image of the real
// unit interval `[0, 1]` under a continuous injective map `f` with
// `f(0) = f(1)`, using the library's pair type for the plane and an
// informal `continuous_curve` predicate for continuity of the map (the
// library has continuity for functions `Real -> Real` but not yet for
// maps into the plane).  The conclusion — that the complement of the
// image splits into exactly two connected components — is recorded with
// the informal predicate `separates_plane`, since the library's
// connectedness machinery (`analysis/topology/connectedness.ac`) is
// developed for general topological spaces but the plane's topology is
// not yet connected to it.
//
// Proof sketch: the standard proofs show first that the complement has at
// most two components (a point of the complement can be joined to the
// exterior by a path avoiding the curve, e.g. by walking along a ray of
// the lattice of small squares), and then at least two (the winding number
// of the curve around an interior point is nonzero while around an
// exterior point it is zero).  All routes need the plane's topology — the
// fundamental group of the circle, the Schoenflies theorem, or homology —
// which is not yet in the library.

// theorem theorems1000_jordan_curve(f: Real -> (Real, Real)) {
//     continuous_curve(f) and f(Real.0) = f(Real.1) and
//     (forall(x: Real, y: Real) {
//         Real.0 <= x and x < Real.1 and Real.0 <= y and y < Real.1 and
//         x != y implies f(x) != f(y)
//     })
//     implies separates_plane_into_two_components(f)
// }
