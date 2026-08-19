from nat import Nat, from_nat
from real import Real, pi, converges_to
from list import partial

numerals Nat

// Stirling's approximation (Stirling's formula): the factorial grows
// asymptotically like
//     n! ~ sqrt(2 pi n) (n / e)^n,
// in the sense that the ratio of the two sides tends to 1 as n tends to
// infinity:
//     lim_{n -> infinity} n! / (sqrt(2 pi n) (n/e)^n) = 1.
// The formula is due to Abraham de Moivre (1730) and James Stirling (1730);
// it is the leading term of an asymptotic expansion whose later terms
// involve the Bernoulli numbers:
//     n! = sqrt(2 pi n) (n/e)^n (1 + 1/(12 n) + 1/(288 n^2) - ...).
// It is the starting point of the elementary proof of the prime number
// theorem (via the Chebyshev estimates) and of the central limit theorem.
//
// The library has the factorial on `Nat`, the real exponential `Real.exp` and
// the constant `pi`.  The square root on reals is available as
// `sqrt(x): Option[Real]`; to stay inside plain real arithmetic the formal
// statement below squares both sides of the limit (the ratio tends to 1 iff
// its square does), replacing `sqrt(2 pi n)` by `2 pi n`:
//     lim_{n -> infinity} n!^2 / (2 pi n (n/e)^(2 n)) = 1,
// which reads, with `converges_to` and the partial-free pointwise form,
//     converges_to(f, 1)  where  f(n) = ...
// with `factorial` on naturals embedded by `from_nat[Real]` and
// `e = (Real.1).exp`.
//
// Proof sketch: apply the trapezoidal rule to the concave logarithm, or
// compare `sum_{k=1}^{n} log k` with the integral of `log`:
//     (n!).log = sum_{k=1}^{n} log k = n log n - n + (1/2) (2 pi n).log + o(1),
// where the constant `(2 pi).log` is evaluated from the Wallis product.
// The proof needs the integral test / Euler-Maclaurin machinery, which is
// not yet in the library; the statement is therefore recorded but not
// proved.
//
// theorem theorems1000_stirling(n: Nat) {
//     Nat.0 < n implies
//         converges_to(
//             function(k: Nat) {
//                 let n: Nat = k + Nat.1
//                 let f: Real = from_nat[Real](n.factorial)
//                 let d: Real = from_nat[Real](Nat.2) * pi * from_nat[Real](n) *
//                     (from_nat[Real](n) / (Real.1).exp).pow(Nat.2 * n)
//                 f * f / d
//             },
//             Real.1)
// }
