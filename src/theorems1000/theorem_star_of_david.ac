from combinatorics.interface import choose_add, factorial_nonzero
from nat import Nat, add_sub, add_comm, add_assoc, mul_comm, mul_assoc, mul_cancel_right,
    lt_imp_lte_suc, lt_suc_right, lte_trans, lte_add_left, lte_suc_suc, mul_to_zero, add_imp_sub

numerals Nat

// Star of David theorem (also known as the "hexagon identity") for binomial
// coefficients: in Pascal's triangle, the product of the three entries
// surrounding any entry in one direction equals the product of the three
// entries surrounding it in the other direction.  For the entry `C(a+1, b+1)`
// the identity reads
//     C(a, b) * C(a+1, b+2) * C(a+2, b+1)
//         = C(a, b+1) * C(a+1, b) * C(a+2, b+2)
// whenever `b + 1 <= a` (so that the six entries are inside Pascal's
// triangle).
//
// Proof: write each of the six binomial coefficients in factorial form,
// `C(n, k) * k! * (n-k)! = n!` (the library theorem `choose_add`).  For the
// three entries of the left-hand side the factorial identities read
//     C(a, b) * b! * (a-b)! = a!,
//     C(a+1, b+2) * (b+2)! * (a-b-1)! = (a+1)!,
//     C(a+2, b+1) * (b+1)! * (a+1-b)! = (a+2)!,
// and for the right-hand side
//     C(a, b+1) * (b+1)! * (a-b-1)! = a!,
//     C(a+1, b) * b! * (a+1-b)! = (a+1)!,
//     C(a+2, b+2) * (b+2)! * (a-b)! = (a+2)!.
// Multiplying the left-hand side by the common factorial product
//     F = b! * (b+1)! * (b+2)! * (a-b)! * (a-b-1)! * (a+1-b)!
// makes each side `a! * (a+1)! * (a+2)!`, and cancelling the nonzero `F`
// gives the identity.

/// The factorial form of a binomial coefficient: for `k <= n`,
/// `C(n, k) * k! * (n-k)! = n!`.
theorem star_factorial_form(n: Nat, k: Nat) {
    k <= n implies n.binom(k) * k.factorial * (n - k).factorial = n.factorial
} by {
    if k <= n {
        choose_add(k, n - k)
        (k + (n - k)).binom(k) * k.factorial * (n - k).factorial =
            (k + (n - k)).factorial
        add_sub(n, k)
        n - k + k = n
        add_comm(n - k, k)
        k + (n - k) = n
        n.binom(k) * k.factorial * (n - k).factorial = n.factorial
    }
}

/// `(a + 1) - (b + 2) = a - (b + 1)` for `b + 1 <= a`.
theorem star_sub_one(a: Nat, b: Nat) {
    b + Nat.1 <= a implies a + Nat.1 - (b + Nat.2) = a - (b + Nat.1)
} by {
    if b + Nat.1 <= a {
        // a = (a - (b+1)) + (b+1)
        add_sub(a, b + Nat.1)
        a - (b + Nat.1) + (b + Nat.1) = a
        // a + 1 = (a - (b+1)) + (b+2)
        add_assoc(a - (b + Nat.1), b + Nat.1, Nat.1)
        a - (b + Nat.1) + (b + Nat.1) + Nat.1 =
            a - (b + Nat.1) + ((b + Nat.1) + Nat.1)
        a - (b + Nat.1) + (b + Nat.1 + Nat.1) = a + Nat.1
        add_assoc(b, Nat.1, Nat.1)
        b + Nat.1 + Nat.1 = b + (Nat.1 + Nat.1)
        b + (Nat.1 + Nat.1) = b + Nat.2
        b + Nat.1 + Nat.1 = b + Nat.2
        a - (b + Nat.1) + (b + Nat.2) = a + Nat.1
        add_imp_sub(a - (b + Nat.1), b + Nat.2, a - (b + Nat.1) + (b + Nat.2))
        (a - (b + Nat.1) + (b + Nat.2)) - (b + Nat.2) = a - (b + Nat.1)
        a + Nat.1 - (b + Nat.2) = a - (b + Nat.1)
    }
}

/// `(a + 2) - (b + 1) = (a + 1) - b` for `b + 1 <= a`.
theorem star_sub_two(a: Nat, b: Nat) {
    b + Nat.1 <= a implies a + Nat.2 - (b + Nat.1) = a + Nat.1 - b
} by {
    if b + Nat.1 <= a {
        // b <= a + 1
        lte_trans(b, b + Nat.1, a)
        b <= b + Nat.1
        b + Nat.1 <= a
        b <= a
        lte_trans(b, a, a + Nat.1)
        a <= a + Nat.1
        b <= a + Nat.1
        // a + 1 = (a + 1 - b) + b
        add_sub(a + Nat.1, b)
        a + Nat.1 - b + b = a + Nat.1
        add_assoc(a + Nat.1 - b, b, Nat.1)
        a + Nat.1 - b + b + Nat.1 = a + Nat.1 - b + (b + Nat.1)
        a + Nat.1 - b + (b + Nat.1) = a + Nat.2
        add_imp_sub(a + Nat.1 - b, b + Nat.1, a + Nat.1 - b + (b + Nat.1))
        (a + Nat.1 - b + (b + Nat.1)) - (b + Nat.1) = a + Nat.1 - b
        a + Nat.2 - (b + Nat.1) = a + Nat.1 - b
    }
}

/// `(a + 2) - (b + 2) = a - b` for `b + 1 <= a`.
theorem star_sub_three(a: Nat, b: Nat) {
    b + Nat.1 <= a implies a + Nat.2 - (b + Nat.2) = a - b
} by {
    if b + Nat.1 <= a {
        // b <= a
        lte_trans(b, b + Nat.1, a)
        b <= b + Nat.1
        b + Nat.1 <= a
        b <= a
        // a = (a - b) + b
        add_sub(a, b)
        a - b + b = a
        add_assoc(a - b, b, Nat.2)
        a - b + b + Nat.2 = a - b + (b + Nat.2)
        a - b + (b + Nat.2) = a + Nat.2
        add_imp_sub(a - b, b + Nat.2, a - b + (b + Nat.2))
        (a - b + (b + Nat.2)) - (b + Nat.2) = a - b
        a + Nat.2 - (b + Nat.2) = a - b
    }
}

/// Cancelling a common nonzero factor: `x * f = y * f` with `f != 0` gives
/// `x = y`.
theorem star_cancel(x: Nat, y: Nat, f: Nat) {
    f != Nat.0 and x * f = y * f implies x = y
} by {
    if f != Nat.0 and x * f = y * f {
        mul_comm(x, f)
        x * f = f * x
        mul_comm(y, f)
        y * f = f * y
        f * x = f * y
        mul_cancel_right(f, x, y)
        x = y
    }
}

// Star of David theorem: the two six-term products around an entry of
// Pascal's triangle are equal.
//
// Proof: multiply both sides by the common factorial product
//     f = b! (b+1)! (b+2)! (a-b)! (a-b-1)! (a+1-b)!,
// written below in the grouping `f = (b! (a-b)!) ((b+2)! (a-b-1)!) ((b+1)! (a+1-b)!)`
// so that each factor of the left-hand side pairs with its factorial
// denominator.  The identity `star_perm` rewrites the same six factorial
// factors into the grouping needed for the right-hand side, and the
// regroupings `star_regroup_left` / `star_regroup_right` interleave each
// binomial with its pair of factorials.  The factorial form
// `star_factorial_form` then identifies each bracket with a factorial,
// making both sides `a! (a+1)! (a+2)!`; cancelling the nonzero common
// factor `f` (all six factorials are nonzero) gives the identity.

/// Swapping the second and third group of a product of three pairs:
/// `(a b) (c d) (e f) = (a b) (e f) (c d)`.
theorem star_swap_groups23(a: Nat, b: Nat, c: Nat, d: Nat, e: Nat, f: Nat) {
    (a * b) * (c * d) * (e * f) = (a * b) * (e * f) * (c * d)
} by {
    mul_assoc(a * b, c * d, e * f)
    (a * b) * (c * d) * (e * f) = (a * b) * ((c * d) * (e * f))
    mul_comm(c * d, e * f)
    (c * d) * (e * f) = (e * f) * (c * d)
    (a * b) * ((c * d) * (e * f)) = (a * b) * ((e * f) * (c * d))
    mul_assoc(a * b, e * f, c * d)
    (a * b) * ((e * f) * (c * d)) = (a * b) * (e * f) * (c * d)
}

/// Swapping the second factors of two adjacent pairs:
/// `(a b) (c d) = (a d) (c b)`.
theorem star_swap_pair_middles(a: Nat, b: Nat, c: Nat, d: Nat) {
    (a * b) * (c * d) = (a * d) * (c * b)
} by {
    mul_assoc(a, b, c * d)
    (a * b) * (c * d) = a * (b * (c * d))
    mul_comm(b, c * d)
    b * (c * d) = (c * d) * b
    a * (b * (c * d)) = a * ((c * d) * b)
    mul_assoc(c, d, b)
    (c * d) * b = c * (d * b)
    a * ((c * d) * b) = a * (c * (d * b))
    mul_comm(d, b)
    d * b = b * d
    a * (c * (d * b)) = a * (c * (b * d))
    mul_assoc(c, b, d)
    (c * b) * d = c * (b * d)
    a * (c * (b * d)) = a * ((c * b) * d)
    mul_comm(c * b, d)
    (c * b) * d = d * (c * b)
    a * ((c * b) * d) = a * (d * (c * b))
    mul_assoc(a, d, c * b)
    (a * d) * (c * b) = a * (d * (c * b))
    a * (d * (c * b)) = (a * d) * (c * b)
}

/// Substituting an equal pair of tail factors in a three-factor product:
/// `x = y implies b1 b2 b3 x = b1 b2 b3 y`.
theorem star_subst(b1: Nat, b2: Nat, b3: Nat, x: Nat, y: Nat) {
    x = y implies b1 * b2 * b3 * x = b1 * b2 * b3 * y
} by {
    if x = y {
        b1 * b2 * b3 * x = b1 * b2 * b3 * y
    }
}

/// The product of two nonzero naturals is nonzero.
theorem star_mul_nonzero(x: Nat, y: Nat) {
    x != Nat.0 and y != Nat.0 implies x * y != Nat.0
} by {
    if x != Nat.0 and y != Nat.0 {
        if x * y = Nat.0 {
            mul_to_zero(x, y)
            x = Nat.0 or y = Nat.0
            if x = Nat.0 {
                false
            }
            if y = Nat.0 {
                false
            }
            false
        }
        x * y != Nat.0
    }
}

/// The common factorial product in the left grouping equals the same
/// product in the right grouping: `(f1 f4)(f3 f5)(f2 f6) = (f2 f5)(f1 f6)(f3 f4)`.
theorem star_perm(f1: Nat, f2: Nat, f3: Nat, f4: Nat, f5: Nat, f6: Nat) {
    (f1 * f4) * (f3 * f5) * (f2 * f6) = (f2 * f5) * (f1 * f6) * (f3 * f4)
} by {
    // (f1 f4)(f3 f5)(f2 f6) = (f1 f4)(f2 f6)(f3 f5)
    star_swap_groups23(f1, f4, f3, f5, f2, f6)
    (f1 * f4) * (f3 * f5) * (f2 * f6) = (f1 * f4) * (f2 * f6) * (f3 * f5)
    // = (f1 f6)(f2 f4)(f3 f5)
    star_swap_pair_middles(f1, f4, f2, f6)
    (f1 * f4) * (f2 * f6) = (f1 * f6) * (f2 * f4)
    (f1 * f4) * (f2 * f6) * (f3 * f5) = (f1 * f6) * (f2 * f4) * (f3 * f5)
    // = (f1 f6)(f2 f5)(f3 f4)
    star_swap_pair_middles(f2, f4, f3, f5)
    (f2 * f4) * (f3 * f5) = (f2 * f5) * (f3 * f4)
    (f1 * f6) * (f2 * f4) * (f3 * f5) = (f1 * f6) * (f2 * f5) * (f3 * f4)
    // = (f2 f5)(f1 f6)(f3 f4)
    star_swap_groups23(f1, f6, f2, f5, f3, f4)
    (f1 * f6) * (f2 * f5) * (f3 * f4) = (f2 * f5) * (f1 * f6) * (f3 * f4)
}
/// Interleaving the left-hand binomials with their factorial pairs:
/// `b1 b2 b3 (f1 f4) (f3 f5) (f2 f6) = (b1 f1 f4) (b2 f3 f5) (b3 f2 f6)`.
theorem star_regroup_left(b1: Nat, b2: Nat, b3: Nat, f1: Nat, f2: Nat, f3: Nat, f4: Nat, f5: Nat, f6: Nat) {
    b1 * b2 * b3 * (f1 * f4) * (f3 * f5) * (f2 * f6) =
        (b1 * f1 * f4) * (b2 * f3 * f5) * (b3 * f2 * f6)
} by {
    b1 * b2 * b3 * (f1 * f4) * (f3 * f5) * (f2 * f6) =
        b1 * b2 * (f1 * f4) * b3 * (f3 * f5) * (f2 * f6)
    b1 * b2 * (f1 * f4) * b3 * (f3 * f5) * (f2 * f6) =
        b1 * (f1 * f4) * b2 * b3 * (f3 * f5) * (f2 * f6)
    b1 * (f1 * f4) * b2 * b3 * (f3 * f5) * (f2 * f6) =
        b1 * (f1 * f4) * b2 * (f3 * f5) * b3 * (f2 * f6)
    b1 * (f1 * f4) * b2 * (f3 * f5) * b3 * (f2 * f6) =
        (b1 * (f1 * f4)) * (b2 * (f3 * f5)) * (b3 * (f2 * f6))
    (b1 * (f1 * f4)) * (b2 * (f3 * f5)) * (b3 * (f2 * f6)) =
        (b1 * f1 * f4) * (b2 * f3 * f5) * (b3 * f2 * f6)
}

/// Interleaving the right-hand binomials with their factorial pairs:
/// `b1 b2 b3 (f2 f5) (f1 f6) (f3 f4) = (b1 f2 f5) (b2 f1 f6) (b3 f3 f4)`.
theorem star_regroup_right(b1: Nat, b2: Nat, b3: Nat, f1: Nat, f2: Nat, f3: Nat, f4: Nat, f5: Nat, f6: Nat) {
    b1 * b2 * b3 * (f2 * f5) * (f1 * f6) * (f3 * f4) =
        (b1 * f2 * f5) * (b2 * f1 * f6) * (b3 * f3 * f4)
} by {
    b1 * b2 * b3 * (f2 * f5) * (f1 * f6) * (f3 * f4) =
        b1 * b2 * (f2 * f5) * b3 * (f1 * f6) * (f3 * f4)
    b1 * b2 * (f2 * f5) * b3 * (f1 * f6) * (f3 * f4) =
        b1 * (f2 * f5) * b2 * b3 * (f1 * f6) * (f3 * f4)
    b1 * (f2 * f5) * b2 * b3 * (f1 * f6) * (f3 * f4) =
        b1 * (f2 * f5) * b2 * (f1 * f6) * b3 * (f3 * f4)
    b1 * (f2 * f5) * b2 * (f1 * f6) * b3 * (f3 * f4) =
        (b1 * (f2 * f5)) * (b2 * (f1 * f6)) * (b3 * (f3 * f4))
    (b1 * (f2 * f5)) * (b2 * (f1 * f6)) * (b3 * (f3 * f4)) =
        (b1 * f2 * f5) * (b2 * f1 * f6) * (b3 * f3 * f4)
}

/// Star of David theorem: for `b + 1 <= a` the two products of three
/// surrounding binomial coefficients are equal:
/// `C(a, b) * C(a+1, b+2) * C(a+2, b+1) =
///      C(a, b+1) * C(a+1, b) * C(a+2, b+2)`.
theorem theorems1000_star_of_david(a: Nat, b: Nat) {
    b + Nat.1 <= a implies
    a.binom(b) * a.suc.binom(b.suc.suc) * a.suc.suc.binom(b.suc) =
        a.binom(b.suc) * a.suc.binom(b) * a.suc.suc.binom(b.suc.suc)
} by {
    if b + Nat.1 <= a {
        // derived bounds
        b <= b + Nat.1
        b + Nat.1 <= a
        lte_trans(b, b + Nat.1, a)
        b <= a
        b.suc = b + Nat.1
        b.suc <= a
        lte_suc_suc(b.suc, a)
        b.suc.suc <= a.suc
        lte_suc_suc(b, a)
        b.suc <= a.suc
        a.suc <= a.suc.suc
        lte_trans(b.suc, a.suc, a.suc.suc)
        b.suc <= a.suc.suc
        lte_trans(b.suc.suc, a.suc, a.suc.suc)
        b.suc.suc <= a.suc.suc
        // the common factorial product, grouped for the left-hand side
        let f: Nat =
            (b.factorial * (a - b).factorial) *
            (b.suc.suc.factorial * (a - (b + Nat.1)).factorial) *
            (b.suc.factorial * (a + Nat.1 - b).factorial)
        // left-hand side times f: interleave each binomial with its pair
        star_regroup_left(a.binom(b), a.suc.binom(b.suc.suc), a.suc.suc.binom(b.suc),
            b.factorial, b.suc.factorial, b.suc.suc.factorial,
            (a - b).factorial, (a - (b + Nat.1)).factorial, (a + Nat.1 - b).factorial)
        a.binom(b) * a.suc.binom(b.suc.suc) * a.suc.suc.binom(b.suc) * f =
            (a.binom(b) * b.factorial * (a - b).factorial) *
            (a.suc.binom(b.suc.suc) * b.suc.suc.factorial *
                (a - (b + Nat.1)).factorial) *
            (a.suc.suc.binom(b.suc) * b.suc.factorial *
                (a + Nat.1 - b).factorial)
        // each bracket is a factorial
        star_factorial_form(a, b)
        a.binom(b) * b.factorial * (a - b).factorial = a.factorial
        star_sub_one(a, b)
        a + Nat.1 - (b + Nat.2) = a - (b + Nat.1)
        b.suc.suc = b + Nat.2
        a.suc - b.suc.suc = a - (b + Nat.1)
        star_factorial_form(a.suc, b.suc.suc)
        a.suc.binom(b.suc.suc) * b.suc.suc.factorial *
            (a.suc - b.suc.suc).factorial = a.suc.factorial
        a.suc.binom(b.suc.suc) * b.suc.suc.factorial *
            (a - (b + Nat.1)).factorial = a.suc.factorial
        star_sub_two(a, b)
        a + Nat.2 - (b + Nat.1) = a + Nat.1 - b
        a + Nat.1 = a.suc
        a + Nat.2 = a.suc.suc
        a.suc.suc - (b + Nat.1) = a.suc - b
        b.suc = b + Nat.1
        a.suc.suc - b.suc = a.suc - b
        star_factorial_form(a.suc.suc, b.suc)
        a.suc.suc.binom(b.suc) * b.suc.factorial *
            (a.suc.suc - b.suc).factorial = a.suc.suc.factorial
        a.suc.suc.binom(b.suc) * b.suc.factorial *
            (a.suc - b).factorial = a.suc.suc.factorial
        a.suc.suc.binom(b.suc) * b.suc.factorial *
            (a + Nat.1 - b).factorial = a.suc.suc.factorial
        (a.binom(b) * b.factorial * (a - b).factorial) *
            (a.suc.binom(b.suc.suc) * b.suc.suc.factorial *
                (a - (b + Nat.1)).factorial) *
            (a.suc.suc.binom(b.suc) * b.suc.factorial *
                (a + Nat.1 - b).factorial) =
            a.factorial * a.suc.factorial * a.suc.suc.factorial
        a.binom(b) * a.suc.binom(b.suc.suc) * a.suc.suc.binom(b.suc) * f =
            a.factorial * a.suc.factorial * a.suc.suc.factorial
        // right-hand side times f: the same six factors regrouped, then interleaved
        star_perm(b.factorial, b.suc.factorial, b.suc.suc.factorial,
            (a - b).factorial, (a - (b + Nat.1)).factorial, (a + Nat.1 - b).factorial)
        (b.factorial * (a - b).factorial) * (b.suc.suc.factorial * (a - (b + Nat.1)).factorial) *
            (b.suc.factorial * (a + Nat.1 - b).factorial) =
            (b.suc.factorial * (a - (b + Nat.1)).factorial) *
            (b.factorial * (a + Nat.1 - b).factorial) *
            (b.suc.suc.factorial * (a - b).factorial)
        a.binom(b.suc) * a.suc.binom(b) * a.suc.suc.binom(b.suc.suc) * f =
            a.binom(b.suc) * a.suc.binom(b) * a.suc.suc.binom(b.suc.suc) *
            ((b.factorial * (a - b).factorial) * (b.suc.suc.factorial * (a - (b + Nat.1)).factorial) *
                (b.suc.factorial * (a + Nat.1 - b).factorial))
        star_subst(a.binom(b.suc), a.suc.binom(b), a.suc.suc.binom(b.suc.suc),
            (b.factorial * (a - b).factorial) * (b.suc.suc.factorial * (a - (b + Nat.1)).factorial) *
                (b.suc.factorial * (a + Nat.1 - b).factorial),
            (b.suc.factorial * (a - (b + Nat.1)).factorial) *
                (b.factorial * (a + Nat.1 - b).factorial) *
                (b.suc.suc.factorial * (a - b).factorial))
        a.binom(b.suc) * a.suc.binom(b) * a.suc.suc.binom(b.suc.suc) *
            ((b.factorial * (a - b).factorial) * (b.suc.suc.factorial * (a - (b + Nat.1)).factorial) *
                (b.suc.factorial * (a + Nat.1 - b).factorial)) =
            a.binom(b.suc) * a.suc.binom(b) * a.suc.suc.binom(b.suc.suc) *
            ((b.suc.factorial * (a - (b + Nat.1)).factorial) *
                (b.factorial * (a + Nat.1 - b).factorial) *
                (b.suc.suc.factorial * (a - b).factorial))
        a.binom(b.suc) * a.suc.binom(b) * a.suc.suc.binom(b.suc.suc) *
            ((b.suc.factorial * (a - (b + Nat.1)).factorial) *
                (b.factorial * (a + Nat.1 - b).factorial) *
                (b.suc.suc.factorial * (a - b).factorial)) =
            a.binom(b.suc) * a.suc.binom(b) * a.suc.suc.binom(b.suc.suc) *
            (b.suc.factorial * (a - (b + Nat.1)).factorial) *
            (b.factorial * (a + Nat.1 - b).factorial) *
            (b.suc.suc.factorial * (a - b).factorial)
        star_regroup_right(a.binom(b.suc), a.suc.binom(b), a.suc.suc.binom(b.suc.suc),
            b.factorial, b.suc.factorial, b.suc.suc.factorial,
            (a - b).factorial, (a - (b + Nat.1)).factorial, (a + Nat.1 - b).factorial)
        a.binom(b.suc) * a.suc.binom(b) * a.suc.suc.binom(b.suc.suc) *
            (b.suc.factorial * (a - (b + Nat.1)).factorial) *
            (b.factorial * (a + Nat.1 - b).factorial) *
            (b.suc.suc.factorial * (a - b).factorial) =
            (a.binom(b.suc) * b.suc.factorial * (a - (b + Nat.1)).factorial) *
            (a.suc.binom(b) * b.factorial * (a + Nat.1 - b).factorial) *
            (a.suc.suc.binom(b.suc.suc) * b.suc.suc.factorial * (a - b).factorial)
        // each bracket is a factorial
        star_sub_one(a, b)
        a + Nat.1 - (b + Nat.2) = a - (b + Nat.1)
        b.suc = b + Nat.1
        b.suc.suc = b + Nat.2
        a - b.suc = a - (b + Nat.1)
        star_factorial_form(a, b.suc)
        a.binom(b.suc) * b.suc.factorial * (a - b.suc).factorial = a.factorial
        a.binom(b.suc) * b.suc.factorial * (a - (b + Nat.1)).factorial = a.factorial
        star_factorial_form(a.suc, b)
        a.suc.binom(b) * b.factorial * (a.suc - b).factorial = a.suc.factorial
        a + Nat.1 = a.suc
        a.suc - b = a + Nat.1 - b
        a.suc.binom(b) * b.factorial * (a + Nat.1 - b).factorial = a.suc.factorial
        star_sub_three(a, b)
        a + Nat.2 - (b + Nat.2) = a - b
        a + Nat.2 = a.suc.suc
        b.suc.suc = b + Nat.2
        a.suc.suc - b.suc.suc = a - b
        star_factorial_form(a.suc.suc, b.suc.suc)
        a.suc.suc.binom(b.suc.suc) * b.suc.suc.factorial *
            (a.suc.suc - b.suc.suc).factorial = a.suc.suc.factorial
        a.suc.suc.binom(b.suc.suc) * b.suc.suc.factorial *
            (a - b).factorial = a.suc.suc.factorial
        (a.binom(b.suc) * b.suc.factorial * (a - (b + Nat.1)).factorial) *
            (a.suc.binom(b) * b.factorial * (a + Nat.1 - b).factorial) *
            (a.suc.suc.binom(b.suc.suc) * b.suc.suc.factorial * (a - b).factorial) =
            a.factorial * a.suc.factorial * a.suc.suc.factorial
        a.binom(b.suc) * a.suc.binom(b) * a.suc.suc.binom(b.suc.suc) * f =
            a.factorial * a.suc.factorial * a.suc.suc.factorial
        // both sides of the identity times f are equal
        a.binom(b) * a.suc.binom(b.suc.suc) * a.suc.suc.binom(b.suc) * f =
            a.binom(b.suc) * a.suc.binom(b) * a.suc.suc.binom(b.suc.suc) * f
        // f != 0: all six factorial factors are nonzero
        factorial_nonzero(b)
        b.factorial != Nat.0
        factorial_nonzero(b.suc)
        b.suc.factorial != Nat.0
        factorial_nonzero(b.suc.suc)
        b.suc.suc.factorial != Nat.0
        factorial_nonzero(a - b)
        (a - b).factorial != Nat.0
        factorial_nonzero(a - (b + Nat.1))
        (a - (b + Nat.1)).factorial != Nat.0
        factorial_nonzero(a + Nat.1 - b)
        (a + Nat.1 - b).factorial != Nat.0
        // each of the three pairs is nonzero
        star_mul_nonzero(b.factorial, (a - b).factorial)
        b.factorial * (a - b).factorial != Nat.0
        star_mul_nonzero(b.suc.suc.factorial, (a - (b + Nat.1)).factorial)
        b.suc.suc.factorial * (a - (b + Nat.1)).factorial != Nat.0
        star_mul_nonzero(b.suc.factorial, (a + Nat.1 - b).factorial)
        b.suc.factorial * (a + Nat.1 - b).factorial != Nat.0
        // and so is the whole product f
        star_mul_nonzero(b.factorial * (a - b).factorial,
            b.suc.suc.factorial * (a - (b + Nat.1)).factorial)
        (b.factorial * (a - b).factorial) *
            (b.suc.suc.factorial * (a - (b + Nat.1)).factorial) != Nat.0
        star_mul_nonzero((b.factorial * (a - b).factorial) *
            (b.suc.suc.factorial * (a - (b + Nat.1)).factorial),
            b.suc.factorial * (a + Nat.1 - b).factorial)
        f != Nat.0
        star_cancel(
            a.binom(b) * a.suc.binom(b.suc.suc) * a.suc.suc.binom(b.suc),
            a.binom(b.suc) * a.suc.binom(b) * a.suc.suc.binom(b.suc.suc),
            f)
        a.binom(b) * a.suc.binom(b.suc.suc) * a.suc.suc.binom(b.suc) =
            a.binom(b.suc) * a.suc.binom(b) * a.suc.suc.binom(b.suc.suc)
    }
}
