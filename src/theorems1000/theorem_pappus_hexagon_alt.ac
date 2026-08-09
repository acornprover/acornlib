from pair import Pair
from real import Real
from top100 import r2_sub
from theorems1000.r2_helpers import r2_cross

// Pappus's hexagon theorem: let `a1`, `a2`, `a3` lie on one line and `b1`,
// `b2`, `b3` on another; then the three intersection points
//     x = a1 b2 and a2 b1,
//     y = a2 b3 and a3 b2,
//     z = a3 b1 and a1 b3
// are collinear.
//
// The two lines are written through the collinearity hypotheses
// `cross(a2 - a1, a3 - a1) = 0` and `cross(b2 - b1, b3 - b1) = 0`, and each
// intersection point is witnessed by two collinearity conditions, e.g. for
// `x` the conditions `cross(x - a1, b2 - a1) = 0` (on `a1 b2`) and
// `cross(x - a2, b1 - a2) = 0` (on `a2 b1`).  The conclusion is the
// collinearity of `x`, `y` and `z`: `cross(z - x, y - x) = 0`.
//
// The statement is commented out because the proof is a long coordinate
// chain: it expands every collinearity hypothesis through the cross-product
// bilinearity lemmas, solves the three intersection points from the six
// collinearity conditions, and verifies the collinearity of the three
// solutions; the sketch is recorded below.
//
// Proof sketch (coordinate form): put the two lines in the position of the
// coordinate axes, `a_i = (alpha_i, 0)` and `b_j = (0, beta_j)`.  The line
// through `a_i` and `b_j` has equation `x / alpha_i + y / beta_j = 1`, and
// the intersection of the lines `a_i b_j` and `a_k b_l` solves the two
// linear equations, giving coordinates that are products of the `alpha` and
// `beta` parameters.  For the three intersections of Pappus's configuration
// the two coordinates come out as
//     x = (alpha1 alpha2 (beta1 - beta2), beta1 beta2 (alpha2 - alpha1)) / (alpha2 beta1 - alpha1 beta2),
//     y = (alpha2 alpha3 (beta2 - beta3), beta2 beta3 (alpha3 - alpha2)) / (alpha3 beta2 - alpha2 beta3),
//     z = (alpha3 alpha1 (beta3 - beta1), beta3 beta1 (alpha1 - alpha3)) / (alpha1 beta3 - alpha3 beta1),
// and expanding `cross(z - x, y - x)` in these coordinates, all terms cancel
// (each numerator product is a scalar multiple of the same six-term
// expression `alpha1 alpha2 alpha3 beta1 beta2 beta3` times a difference of
// the denominators), so the three points are collinear.  In the vector
// formulation this expansion runs over the cross-product bilinearity lemmas
// and the affine parametrisations of the two lines, which is a long but
// purely algebraic chain.
//
// theorem theorems1000_pappus_hexagon(
//     a1: Pair[Real, Real], a2: Pair[Real, Real], a3: Pair[Real, Real],
//     b1: Pair[Real, Real], b2: Pair[Real, Real], b3: Pair[Real, Real],
//     x: Pair[Real, Real], y: Pair[Real, Real], z: Pair[Real, Real]
// ) {
//     r2_cross(r2_sub(a2, a1), r2_sub(a3, a1)) = Real.0 and
//     r2_cross(r2_sub(b2, b1), r2_sub(b3, b1)) = Real.0 and
//     r2_cross(r2_sub(x, a1), r2_sub(b2, a1)) = Real.0 and
//     r2_cross(r2_sub(x, a2), r2_sub(b1, a2)) = Real.0 and
//     r2_cross(r2_sub(y, a2), r2_sub(b3, a2)) = Real.0 and
//     r2_cross(r2_sub(y, a3), r2_sub(b2, a3)) = Real.0 and
//     r2_cross(r2_sub(z, a3), r2_sub(b1, a3)) = Real.0 and
//     r2_cross(r2_sub(z, a1), r2_sub(b3, a1)) = Real.0 and
//     r2_cross(r2_sub(b1, a1), r2_sub(a2, a1)) != Real.0 and
//     r2_cross(r2_sub(b2, a1), r2_sub(a3, a1)) != Real.0 and
//     r2_cross(r2_sub(b3, a1), r2_sub(a2, a1)) != Real.0
//     implies
//     r2_cross(r2_sub(z, x), r2_sub(y, x)) = Real.0
// }
