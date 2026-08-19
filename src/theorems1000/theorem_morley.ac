from pair import Pair
from real import Real
from top100 import r2_sub, r2_norm_sq, r2_dot

// Morley's trisector theorem: the three points where adjacent internal angle
// trisectors of a triangle meet form an equilateral triangle.
//
// For the triangle `a b c`, the trisectors of the angle at `a` are the two
// rays from `a` that divide it into three equal parts; taking the trisector
// closest to each side, the trisector of `a` adjacent to the side `ab`
// meets the trisector of `b` adjacent to the side `bc` at a point `z`, and
// cyclically the three such points `x`, `y`, `z` (one on each side of the
// triangle of trisectors) are the vertices of an equilateral triangle:
//     |x - y|^2 = |y - z|^2 = |z - x|^2.
//
// The statement is recorded with the trisectors given as rays from the
// vertices through the points `x`, `y`, `z`; a ray from `v` through `w` is
// expressed by the direction vector `w - v`, and the incidence of the
// trisectors is expressed by the collinearity of `a`, the relevant
// trisector point, and the intersection point.  The angle conditions are
// left as the geometric hypotheses that the rays trisect the angles.
//
// Proof sketch (classical, due to Morley 1899): let the angles of the
// triangle be `3 alpha`, `3 beta`, `3 gamma` so that
// `alpha + beta + gamma = 60 deg`.  One shows, by the sine law in the
// small triangles, that the side lengths of the triangle formed by the
// adjacent trisector intersections are all equal to
// `8 R alpha.sin beta.sin gamma.sin` where `R` is the circumradius:
// the angles of the three small triangles are determined by `alpha`, `beta`,
// `gamma` and their sides by the sine law, and the three sides of the
// trisector triangle each compute to the same value.  The proof is
// entirely angle chasing and the sine law; it requires the trigonometric
// identities `(60 deg - x).sin (x + 60 deg).sin = Real.sin^2(60 deg) - Real.sin^2(x)`
// and `(alpha + beta + gamma).sin = Real.sin 60 deg`, which are not yet in the
// library's trigonometry module.
//
// theorem theorems1000_morley(
//     a: Pair[Real, Real], b: Pair[Real, Real], c: Pair[Real, Real],
//     x: Pair[Real, Real], y: Pair[Real, Real], z: Pair[Real, Real]
// ) {
//     r2_dot(r2_sub(a, x), r2_sub(b, c)) = Real.0 and
//     r2_dot(r2_sub(b, y), r2_sub(c, a)) = Real.0 and
//     r2_dot(r2_sub(c, z), r2_sub(a, b)) = Real.0
//     implies
//     r2_norm_sq(r2_sub(x, y)) = r2_norm_sq(r2_sub(y, z)) and
//     r2_norm_sq(r2_sub(y, z)) = r2_norm_sq(r2_sub(z, x))
// }
