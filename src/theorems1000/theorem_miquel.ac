from pair import Pair
from real import Real
from top100 import r2_add, r2_sub, r2_norm_sq
from theorems1000.r2_helpers import r2_smul

// Miquel's theorem: in a triangle, a point chosen on each side gives three
// circles — through a vertex and the two adjacent chosen points — and these
// three circles pass through a common point, the Miquel point.
//
// Concretely: with the triangle `abc`, a point `x` on the side `bc`, a point
// `y` on `ca` and a point `z` on `ab` (written as affine combinations with
// parameters `s`, `t`, `u`), the three circles
//     (a y z),   (b z x),   (c x y)
// share a common point `m`.  The statement below asserts the existence of a
// common point `m` together with centers `o1`, `o2`, `o3` and squared radii
// `r1`, `r2`, `r3` witnessing that `m` lies on each of the three circles,
// where "on the circle with center `o` and squared radius `r`" means having
// squared distance `r` from `o` (so each circle is determined by the three
// points named in its parentheses).
//
// Proof sketch (angle chasing): the four points `a`, `y`, `z`, `m` are
// concyclic and `b`, `z`, `x`, `m` are concyclic, so the inscribed angles
// give `angle(ymz) = angle(yaz)` and `angle(zmx) = angle(zbx)`; since
// `x`, `y`, `z` lie on the sides of the triangle, the three angles
// `angle(yaz)`, `angle(zbx)` and `angle(xcy)` sum to 180 degrees, which
// forces `angle(ymx) = 180 - angle(ycx)`, i.e. `c`, `x`, `y`, `m` are
// concyclic as well.  In coordinates the circle conditions are four equal
// squared distances per circle; the prover-visible proof would solve for the
// circle through three points (the center is obtained by intersecting the
// perpendicular bisectors, i.e. from the two linear equations in the center
// coordinates given by equal squared distances) and verify the fourth equal
// distance, expanding with `r2_norm_sq_add_expansion` and the scalar laws in
// `r2_helpers`.  The statement is recorded with the affine parameters and
// the witness circles.
//
// theorem theorems1000_miquel(
//     a: Pair[Real, Real], b: Pair[Real, Real], c: Pair[Real, Real],
//     x: Pair[Real, Real], y: Pair[Real, Real], z: Pair[Real, Real],
//     s: Real, t: Real, u: Real,
//     m: Pair[Real, Real],
//     o1: Pair[Real, Real], o2: Pair[Real, Real], o3: Pair[Real, Real],
//     r1: Real, r2: Real, r3: Real
// ) {
//     r2_add(b, r2_smul(s, r2_sub(c, b))) = x and
//     r2_add(c, r2_smul(t, r2_sub(a, c))) = y and
//     r2_add(a, r2_smul(u, r2_sub(b, a))) = z and
//     r2_norm_sq(r2_sub(a, o1)) = r1 and r2_norm_sq(r2_sub(y, o1)) = r1 and
//     r2_norm_sq(r2_sub(z, o1)) = r1 and r2_norm_sq(r2_sub(m, o1)) = r1 and
//     r2_norm_sq(r2_sub(b, o2)) = r2 and r2_norm_sq(r2_sub(z, o2)) = r2 and
//     r2_norm_sq(r2_sub(x, o2)) = r2 and r2_norm_sq(r2_sub(m, o2)) = r2 and
//     r2_norm_sq(r2_sub(c, o3)) = r3 and r2_norm_sq(r2_sub(x, o3)) = r3 and
//     r2_norm_sq(r2_sub(y, o3)) = r3 and r2_norm_sq(r2_sub(m, o3)) = r3
// }
