from pair import Pair
from real import Real
from top100 import r2_sub, r2_norm_sq, r2_dot

// Carnot's theorem: for a triangle `a b c` and points `x` on the side
// `bc`, `y` on `ca`, `z` on `ab`, the perpendiculars `ax`, `by`, `cz` are
// concurrent if and only if
//     |b - x|^2 - |c - x|^2 + |c - y|^2 - |a - y|^2 + |a - z|^2 - |b - z|^2 = 0.
// (This is the theorem of Lazare Carnot about perpendiculars to the sides
// of a triangle, not to be confused with Carnot's theorem on heat engines.)
//
// The statement is recorded with `x` on the line `bc` (the collinearity
// `(x - b) . (c - b) = |c - b|^2` being replaced by the affine form
// `x = b + s (c - b)` for a real parameter `s`, written in the hypothesis
// through the defining equation of the parameter), the perpendiculars
// `(a - x) . (c - b) = 0` and cyclically, and the concurrence witness `p`
// with `p` on each of the three lines (`(p - a) . (x - a) = 0` type
// hypotheses).  The conclusion is the sum of squared-length differences
// above.
//
// Proof sketch: choose coordinates with the three side lines, and write
// the squared distances in terms of the parameters `s`, `t`, `u` of `x`,
// `y`, `z` along the sides and the side lengths.  The condition that the
// three perpendiculars meet at a point `p` is equivalent, by solving the
// three perpendicularity equations for the coordinates of `p`, to the
// vanishing of the alternating sum of squared distances; the verification
// is an expansion of the squared distances along the parameterized sides
// and the cancellation of the parameter-dependent terms, exactly the
// coordinate computation of the classical proof.
//
// theorem theorems1000_carnot(
//     a: Pair[Real, Real], b: Pair[Real, Real], c: Pair[Real, Real],
//     x: Pair[Real, Real], y: Pair[Real, Real], z: Pair[Real, Real],
//     p: Pair[Real, Real]
// ) {
//     r2_dot(r2_sub(a, x), r2_sub(c, b)) = Real.0 and
//     r2_dot(r2_sub(b, y), r2_sub(a, c)) = Real.0 and
//     r2_dot(r2_sub(c, z), r2_sub(b, a)) = Real.0 and
//     r2_dot(r2_sub(p, a), r2_sub(x, a)) = Real.0 and
//     r2_dot(r2_sub(p, b), r2_sub(y, b)) = Real.0 and
//     r2_dot(r2_sub(p, c), r2_sub(z, c)) = Real.0
//     implies
//     r2_norm_sq(r2_sub(b, x)) - r2_norm_sq(r2_sub(c, x)) +
//         r2_norm_sq(r2_sub(c, y)) - r2_norm_sq(r2_sub(a, y)) +
//         r2_norm_sq(r2_sub(a, z)) - r2_norm_sq(r2_sub(b, z)) = Real.0
// }
