from real import Real
from theorems1000.r2_helpers import real_add_pair_rearrange
from theorems1000.ring_helpers import ring_square_add, ring_square_sub,
    ring_cancel_middle, ring_cancel_middle_plus

numerals Real

// Lagrange's identity (three-dimensional form): for real numbers
// `a, b, c, d, e, f`,
//     (a^2 + b^2 + c^2) (d^2 + e^2 + f^2)
//         = (a d + b e + c f)^2 + (a e - b d)^2 + (a f - c d)^2 + (b f - c e)^2.
// In vector form this is the identity `|u|^2 |v|^2 = (u . v)^2 + |u x v|^2`
// for the cross product, so it is the algebraic content of the
// Cauchy-Schwarz inequality in three dimensions.  The two-dimensional
// scalar case is the Brahmagupta-Fibonacci identity (proved in
// theorem_brahmagupta_fibonacci.ac in this directory, and also as a
// generic lemma in geometry/point2_heron.ac), and the four-dimensional
// case is Euler's four-square identity (theorem_lagrange_four_squares.ac
// in this directory).
//
// Proof sketch: expanding the three-term square and the three difference
// squares gives the nine pure terms
//     a^2 d^2 + b^2 e^2 + c^2 f^2 + a^2 e^2 + b^2 d^2
//         + a^2 f^2 + c^2 d^2 + b^2 f^2 + c^2 e^2
// plus the six cross terms
//     +2 a b d e - 2 a b d e + 2 a c d f - 2 a c d f + 2 b c e f - 2 b c e f,
// which cancel pairwise (each pair is four copies of the same monomial,
// two positive and two negative).  The nine pure terms are exactly the
// expansion of `(a^2 + b^2 + c^2)(d^2 + e^2 + f^2)`.
//
// The proof below expands the four squares with the ring_helpers
// machinery, cancels each pair of opposite cross terms with the
// `ring_cancel_middle` / `ring_cancel_middle_plus` lemmas, and
// reassembles the pure terms.

/// The square of a three-term sum expands into the nine ordered products.
theorem lg_square_three(x: Real, y: Real, z: Real) {
    (x + y + z) * (x + y + z) =
        x * x + y * x + z * x + (x * y + y * y + z * y + (x * z + y * z + z * z))
} by {
    ring_square_add(x + y, z)
    ((x + y) + z) * ((x + y) + z) = (x + y) * (x + y) + z * (x + y) + ((x + y) * z + z * z)
    ring_square_add(x, y)
    (x + y) * (x + y) = x * x + y * x + (x * y + y * y)
    ((x + y) + z) * ((x + y) + z) =
        x * x + y * x + (x * y + y * y) + z * (x + y) + ((x + y) * z + z * z)
    z * (x + y) = z * x + z * y
    (x + y) * z = x * z + y * z
    ((x + y) + z) * ((x + y) + z) =
        x * x + y * x + (x * y + y * y) + (z * x + z * y) + ((x * z + y * z) + z * z)
    ((x + y) + z) * ((x + y) + z) =
        x * x + y * x + z * x + (x * y + y * y + z * y + (x * z + y * z + z * z))
    (x + y + z) * (x + y + z) =
        x * x + y * x + z * x + (x * y + y * y + z * y + (x * z + y * z + z * z))
}

/// Swapping the second and third summands of a four-term sum.
theorem lg_swap23(a: Real, b: Real, c: Real, d: Real) {
    a + b + c + d = a + c + b + d
} by {
    real_add_pair_rearrange(a, b, c, d)
    (a + b) + (c + d) = (a + c) + (b + d)
    a + b + c + d = a + c + b + d
}

/// The nine pure terms of the right-hand side, reordered into the order
/// of the left-hand side expansion.
theorem lg_pure_reassemble(a: Real, b: Real, c: Real, d: Real, e: Real, f: Real) {
    a * d * a * d + b * e * b * e + c * f * c * f +
    a * e * a * e + b * d * b * d + a * f * a * f + c * d * c * d +
    b * f * b * f + c * e * c * e =
        a * a * d * d + a * a * e * e + a * a * f * f +
        b * b * d * d + b * b * e * e + b * b * f * f +
        c * c * d * d + c * c * e * e + c * c * f * f
} by {
    a * d * a * d = a * a * d * d
    b * e * b * e = b * b * e * e
    c * f * c * f = c * c * f * f
    a * e * a * e = a * a * e * e
    b * d * b * d = b * b * d * d
    a * f * a * f = a * a * f * f
    c * d * c * d = c * c * d * d
    b * f * b * f = b * b * f * f
    c * e * c * e = c * c * e * e
    lg_swap23(a * a * d * d + b * b * e * e, c * c * f * f, a * a * e * e,
        b * b * d * d + a * a * f * f + c * c * d * d + b * b * f * f + c * c * e * e)
    a * a * d * d + b * b * e * e + a * a * e * e + c * c * f * f + b * b * d * d + a * a * f * f + c * c * d * d + b * b * f * f + c * c * e * e =
        a * a * d * d + b * b * e * e + a * a * e * e + c * c * f * f + b * b * d * d + a * a * f * f + c * c * d * d + b * b * f * f + c * c * e * e
    lg_swap23(a * a * d * d, b * b * e * e, a * a * e * e,
        c * c * f * f + b * b * d * d + a * a * f * f + c * c * d * d + b * b * f * f + c * c * e * e)
    a * a * d * d + a * a * e * e + b * b * e * e + c * c * f * f + b * b * d * d + a * a * f * f + c * c * d * d + b * b * f * f + c * c * e * e =
        a * a * d * d + a * a * e * e + b * b * e * e + c * c * f * f + b * b * d * d + a * a * f * f + c * c * d * d + b * b * f * f + c * c * e * e
    lg_swap23(a * a * d * d + a * a * e * e + b * b * e * e + c * c * f * f,
        b * b * d * d, a * a * f * f, c * c * d * d + b * b * f * f + c * c * e * e)
    a * a * d * d + a * a * e * e + b * b * e * e + c * c * f * f + a * a * f * f + b * b * d * d + c * c * d * d + b * b * f * f + c * c * e * e =
        a * a * d * d + a * a * e * e + b * b * e * e + c * c * f * f + a * a * f * f + b * b * d * d + c * c * d * d + b * b * f * f + c * c * e * e
    lg_swap23(a * a * d * d + a * a * e * e + b * b * e * e, c * c * f * f, a * a * f * f,
        b * b * d * d + c * c * d * d + b * b * f * f + c * c * e * e)
    a * a * d * d + a * a * e * e + b * b * e * e + a * a * f * f + c * c * f * f + b * b * d * d + c * c * d * d + b * b * f * f + c * c * e * e =
        a * a * d * d + a * a * e * e + b * b * e * e + a * a * f * f + c * c * f * f + b * b * d * d + c * c * d * d + b * b * f * f + c * c * e * e
    lg_swap23(a * a * d * d + a * a * e * e, b * b * e * e, a * a * f * f,
        c * c * f * f + b * b * d * d + c * c * d * d + b * b * f * f + c * c * e * e)
    a * a * d * d + a * a * e * e + a * a * f * f + b * b * e * e + c * c * f * f + b * b * d * d + c * c * d * d + b * b * f * f + c * c * e * e =
        a * a * d * d + a * a * e * e + a * a * f * f + b * b * e * e + c * c * f * f + b * b * d * d + c * c * d * d + b * b * f * f + c * c * e * e
    lg_swap23(a * a * d * d + a * a * e * e + a * a * f * f + b * b * e * e,
        c * c * f * f, b * b * d * d, c * c * d * d + b * b * f * f + c * c * e * e)
    a * a * d * d + a * a * e * e + a * a * f * f + b * b * e * e + b * b * d * d + c * c * f * f + c * c * d * d + b * b * f * f + c * c * e * e =
        a * a * d * d + a * a * e * e + a * a * f * f + b * b * e * e + b * b * d * d + c * c * f * f + c * c * d * d + b * b * f * f + c * c * e * e
    lg_swap23(a * a * d * d + a * a * e * e + a * a * f * f,
        b * b * e * e, b * b * d * d, c * c * f * f + c * c * d * d + b * b * f * f + c * c * e * e)
    a * a * d * d + a * a * e * e + a * a * f * f + b * b * d * d + b * b * e * e + c * c * f * f + c * c * d * d + b * b * f * f + c * c * e * e =
        a * a * d * d + a * a * e * e + a * a * f * f + b * b * d * d + b * b * e * e + c * c * f * f + c * c * d * d + b * b * f * f + c * c * e * e
    lg_swap23(a * a * d * d + a * a * e * e + a * a * f * f + b * b * d * d + b * b * e * e + c * c * f * f,
        c * c * d * d, b * b * f * f, c * c * e * e)
    a * a * d * d + a * a * e * e + a * a * f * f + b * b * d * d + b * b * e * e + c * c * f * f + b * b * f * f + c * c * d * d + c * c * e * e =
        a * a * d * d + a * a * e * e + a * a * f * f + b * b * d * d + b * b * e * e + c * c * f * f + b * b * f * f + c * c * d * d + c * c * e * e
    lg_swap23(a * a * d * d + a * a * e * e + a * a * f * f + b * b * d * d + b * b * e * e,
        c * c * f * f, b * b * f * f, c * c * d * d + c * c * e * e)
    a * a * d * d + a * a * e * e + a * a * f * f + b * b * d * d + b * b * e * e + b * b * f * f + c * c * f * f + c * c * d * d + c * c * e * e =
        a * a * d * d + a * a * e * e + a * a * f * f + b * b * d * d + b * b * e * e + b * b * f * f + c * c * f * f + c * c * d * d + c * c * e * e
    lg_swap23(a * a * d * d + a * a * e * e + a * a * f * f + b * b * d * d + b * b * e * e + b * b * f * f,
        c * c * f * f, c * c * d * d, c * c * e * e)
    a * a * d * d + a * a * e * e + a * a * f * f + b * b * d * d + b * b * e * e + b * b * f * f + c * c * d * d + c * c * f * f + c * c * e * e =
        a * a * d * d + a * a * e * e + a * a * f * f + b * b * d * d + b * b * e * e + b * b * f * f + c * c * d * d + c * c * f * f + c * c * e * e
    lg_swap23(a * a * d * d + a * a * e * e + a * a * f * f + b * b * d * d + b * b * e * e + b * b * f * f + c * c * d * d,
        c * c * f * f, c * c * e * e, Real.0)
    a * a * d * d + a * a * e * e + a * a * f * f + b * b * d * d + b * b * e * e + b * b * f * f + c * c * d * d + c * c * e * e + c * c * f * f =
        a * a * d * d + a * a * e * e + a * a * f * f + b * b * d * d + b * b * e * e + b * b * f * f + c * c * d * d + c * c * e * e + c * c * f * f
}

/// The left-hand side expands into the nine pure terms, grouped by the
/// left factor (the natural order of the product expansion).
theorem lg_lhs_expand(a: Real, b: Real, c: Real, d: Real, e: Real, f: Real) {
    (a * a + b * b + c * c) * (d * d + e * e + f * f) =
        a * a * d * d + a * a * e * e + a * a * f * f +
        b * b * d * d + b * b * e * e + b * b * f * f +
        c * c * d * d + c * c * e * e + c * c * f * f
} by {
    (a * a + b * b + c * c) * (d * d + e * e + f * f) =
        a * a * (d * d + e * e + f * f) +
        (b * b + c * c) * (d * d + e * e + f * f)
    (b * b + c * c) * (d * d + e * e + f * f) =
        b * b * (d * d + e * e + f * f) + c * c * (d * d + e * e + f * f)
    a * a * (d * d + e * e + f * f) = a * a * (d * d + e * e) + a * a * f * f
    a * a * (d * d + e * e) = a * a * d * d + a * a * e * e
    a * a * (d * d + e * e + f * f) = a * a * d * d + a * a * e * e + a * a * f * f
    b * b * (d * d + e * e + f * f) = b * b * (d * d + e * e) + b * b * f * f
    b * b * (d * d + e * e) = b * b * d * d + b * b * e * e
    b * b * (d * d + e * e + f * f) = b * b * d * d + b * b * e * e + b * b * f * f
    c * c * (d * d + e * e + f * f) = c * c * (d * d + e * e) + c * c * f * f
    c * c * (d * d + e * e) = c * c * d * d + c * c * e * e
    c * c * (d * d + e * e + f * f) = c * c * d * d + c * c * e * e + c * c * f * f
    (a * a + b * b + c * c) * (d * d + e * e + f * f) =
        a * a * d * d + a * a * e * e + a * a * f * f +
        (b * b * d * d + b * b * e * e + b * b * f * f) +
        (c * c * d * d + c * c * e * e + c * c * f * f)
    a * a * d * d + a * a * e * e + a * a * f * f +
        (b * b * d * d + b * b * e * e + b * b * f * f) +
        (c * c * d * d + c * c * e * e + c * c * f * f) =
        a * a * d * d + a * a * e * e + a * a * f * f +
        b * b * d * d + b * b * e * e + b * b * f * f +
        c * c * d * d + c * c * e * e + c * c * f * f
}

// theorem theorems1000_lagrange_identity(
//     a: Real, b: Real, c: Real, d: Real, e: Real, f: Real
// ) {
//     (a * a + b * b + c * c) * (d * d + e * e + f * f) =
//         (a * d + b * e + c * f) * (a * d + b * e + c * f) +
//         (a * e - b * d) * (a * e - b * d) +
//         (a * f - c * d) * (a * f - c * d) +
//         (b * f - c * e) * (b * f - c * e)
// }
