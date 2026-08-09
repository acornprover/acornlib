from pair import Pair
from real import Real
from top100 import r2_add, r2_neg, r2_sub, r2_dot, r2_norm_sq
from theorems1000.r2_helpers import r2_sub_through, r2_dot_add_left, r2_norm_sq_add_add_expand, r2_norm_sq_add_expansion, r2_add_assoc

// The British flag theorem: in a rectangle, the sums of the squared distances
// from an arbitrary point to opposite corners are equal.
//
// The rectangle has corners `a`, `b = a + u`, `c = a + u + v` and
// `d = a + v`; the hypothesis `(b - a) . (d - a) = 0` is the right angle at
// `a`, and `c - a = (b - a) + (d - a)` is the parallelogram law.

/// The British flag theorem for a rectangle in the coordinate plane.
///
/// For the rectangle with corners `a`, `b`, `c` and `d` (with `b - a`
/// perpendicular to `d - a` and `c - a = (b - a) + (d - a)`) and an arbitrary
/// point `p`, the sums of the squared distances from `p` to opposite corners
/// are equal: |p-a|^2 + |p-c|^2 = |p-b|^2 + |p-d|^2.
theorem theorems1000_british_flag(
    a: Pair[Real, Real], b: Pair[Real, Real], c: Pair[Real, Real],
    d: Pair[Real, Real], p: Pair[Real, Real]
) {
    r2_dot(r2_sub(b, a), r2_sub(d, a)) = Real.0 and
    r2_sub(c, a) = r2_add(r2_sub(b, a), r2_sub(d, a)) implies
    r2_norm_sq(r2_sub(a, p)) + r2_norm_sq(r2_sub(c, p)) =
        r2_norm_sq(r2_sub(b, p)) + r2_norm_sq(r2_sub(d, p))
} by {
    if r2_dot(r2_sub(b, a), r2_sub(d, a)) = Real.0 and
        r2_sub(c, a) = r2_add(r2_sub(b, a), r2_sub(d, a)) {
        r2_dot(r2_sub(b, a), r2_sub(d, a)) = Real.0
        r2_sub(c, a) = r2_add(r2_sub(b, a), r2_sub(d, a))
        // w = a - p, u = b - a, v = d - a; the four displacements from p:
        // b - p = w + u, d - p = w + v, c - p = (w + u) + v
        r2_sub_through(p, a, b)
        r2_sub(b, p) = r2_add(r2_sub(a, p), r2_sub(b, a))
        r2_sub_through(p, a, d)
        r2_sub(d, p) = r2_add(r2_sub(a, p), r2_sub(d, a))
        r2_sub_through(p, a, c)
        r2_sub(c, p) = r2_add(r2_sub(a, p), r2_sub(c, a))
        r2_sub(c, p) = r2_add(r2_sub(a, p), r2_add(r2_sub(b, a), r2_sub(d, a)))
        r2_add_assoc(r2_sub(a, p), r2_sub(b, a), r2_sub(d, a))
        r2_sub(c, p) = r2_add(r2_add(r2_sub(a, p), r2_sub(b, a)), r2_sub(d, a))
        // expansions
        r2_norm_sq_add_expansion(r2_sub(a, p), r2_sub(b, a))
        r2_norm_sq(r2_sub(b, p)) =
            r2_norm_sq(r2_sub(a, p)) + r2_norm_sq(r2_sub(b, a)) +
            r2_dot(r2_sub(a, p), r2_sub(b, a)) + r2_dot(r2_sub(a, p), r2_sub(b, a))
        r2_norm_sq_add_expansion(r2_sub(a, p), r2_sub(d, a))
        r2_norm_sq(r2_sub(d, p)) =
            r2_norm_sq(r2_sub(a, p)) + r2_norm_sq(r2_sub(d, a)) +
            r2_dot(r2_sub(a, p), r2_sub(d, a)) + r2_dot(r2_sub(a, p), r2_sub(d, a))
        r2_norm_sq_add_add_expand(r2_sub(a, p), r2_sub(b, a), r2_sub(d, a))
        r2_norm_sq(r2_sub(c, p)) =
            r2_norm_sq(r2_sub(a, p)) + r2_norm_sq(r2_sub(b, a)) + r2_norm_sq(r2_sub(d, a)) +
            r2_dot(r2_sub(a, p), r2_sub(b, a)) + r2_dot(r2_sub(a, p), r2_sub(b, a)) +
            r2_dot(r2_sub(a, p), r2_sub(d, a)) + r2_dot(r2_sub(a, p), r2_sub(d, a)) +
            r2_dot(r2_sub(b, a), r2_sub(d, a)) + r2_dot(r2_sub(b, a), r2_sub(d, a))
        // (w + u) . v = w . v + u . v = w . v
        r2_dot_add_left(r2_sub(a, p), r2_sub(b, a), r2_sub(d, a))
        r2_dot(r2_add(r2_sub(a, p), r2_sub(b, a)), r2_sub(d, a)) =
            r2_dot(r2_sub(a, p), r2_sub(d, a)) + r2_dot(r2_sub(b, a), r2_sub(d, a))
        // |w + u + v|^2 = |w + u|^2 + |v|^2 + 2 (w+u) . v
        r2_norm_sq_add_expansion(r2_add(r2_sub(a, p), r2_sub(b, a)), r2_sub(d, a))
        r2_norm_sq(r2_add(r2_add(r2_sub(a, p), r2_sub(b, a)), r2_sub(d, a))) =
            r2_norm_sq(r2_add(r2_sub(a, p), r2_sub(b, a))) + r2_norm_sq(r2_sub(d, a)) +
            r2_dot(r2_add(r2_sub(a, p), r2_sub(b, a)), r2_sub(d, a)) +
            r2_dot(r2_add(r2_sub(a, p), r2_sub(b, a)), r2_sub(d, a))
        r2_dot(r2_add(r2_sub(a, p), r2_sub(b, a)), r2_sub(d, a)) =
            r2_dot(r2_sub(a, p), r2_sub(d, a)) + r2_dot(r2_sub(b, a), r2_sub(d, a))
        r2_dot(r2_sub(b, a), r2_sub(d, a)) = Real.0
        // (w + u) . v = w . v, since u . v = 0
        r2_dot(r2_add(r2_sub(a, p), r2_sub(b, a)), r2_sub(d, a)) =
            r2_dot(r2_sub(a, p), r2_sub(d, a)) + r2_dot(r2_sub(b, a), r2_sub(d, a))
        r2_dot(r2_sub(a, p), r2_sub(d, a)) + r2_dot(r2_sub(b, a), r2_sub(d, a)) =
            r2_dot(r2_sub(a, p), r2_sub(d, a)) + Real.0
        r2_dot(r2_sub(a, p), r2_sub(d, a)) + Real.0 = r2_dot(r2_sub(a, p), r2_sub(d, a))
        r2_dot(r2_add(r2_sub(a, p), r2_sub(b, a)), r2_sub(d, a)) =
            r2_dot(r2_sub(a, p), r2_sub(d, a))
        // |c - p|^2 = |w + u|^2 + |v|^2 + 2 (w . v)
        r2_norm_sq(r2_add(r2_add(r2_sub(a, p), r2_sub(b, a)), r2_sub(d, a))) =
            r2_norm_sq(r2_add(r2_sub(a, p), r2_sub(b, a))) + r2_norm_sq(r2_sub(d, a)) +
            r2_dot(r2_add(r2_sub(a, p), r2_sub(b, a)), r2_sub(d, a)) +
            r2_dot(r2_add(r2_sub(a, p), r2_sub(b, a)), r2_sub(d, a))
        r2_norm_sq(r2_add(r2_add(r2_sub(a, p), r2_sub(b, a)), r2_sub(d, a))) =
            r2_norm_sq(r2_add(r2_sub(a, p), r2_sub(b, a))) + r2_norm_sq(r2_sub(d, a)) +
            r2_dot(r2_sub(a, p), r2_sub(d, a)) + r2_dot(r2_sub(a, p), r2_sub(d, a))
        r2_norm_sq(r2_sub(c, p)) =
            r2_norm_sq(r2_add(r2_sub(a, p), r2_sub(b, a))) + r2_norm_sq(r2_sub(d, a)) +
            r2_dot(r2_sub(a, p), r2_sub(d, a)) + r2_dot(r2_sub(a, p), r2_sub(d, a))
        // |b - p|^2 = |w + u|^2
        r2_norm_sq_add_expansion(r2_sub(a, p), r2_sub(b, a))
        r2_norm_sq(r2_add(r2_sub(a, p), r2_sub(b, a))) =
            r2_norm_sq(r2_sub(a, p)) + r2_norm_sq(r2_sub(b, a)) +
            r2_dot(r2_sub(a, p), r2_sub(b, a)) + r2_dot(r2_sub(a, p), r2_sub(b, a))
        r2_norm_sq(r2_sub(b, p)) = r2_norm_sq(r2_add(r2_sub(a, p), r2_sub(b, a)))
        // assemble: |w|^2 + |c-p|^2 = |w+u|^2 + |w|^2 + |v|^2 + 2 (w . v) = |b-p|^2 + |d-p|^2
        r2_norm_sq(r2_sub(a, p)) + r2_norm_sq(r2_sub(c, p)) =
            r2_norm_sq(r2_sub(a, p)) +
            (r2_norm_sq(r2_add(r2_sub(a, p), r2_sub(b, a))) + r2_norm_sq(r2_sub(d, a)) +
            r2_dot(r2_sub(a, p), r2_sub(d, a)) + r2_dot(r2_sub(a, p), r2_sub(d, a)))
        r2_norm_sq(r2_sub(b, p)) + r2_norm_sq(r2_sub(d, p)) =
            r2_norm_sq(r2_add(r2_sub(a, p), r2_sub(b, a))) + r2_norm_sq(r2_sub(d, p))
        r2_norm_sq(r2_add(r2_sub(a, p), r2_sub(b, a))) + r2_norm_sq(r2_sub(d, p)) =
            r2_norm_sq(r2_add(r2_sub(a, p), r2_sub(b, a))) +
            (r2_norm_sq(r2_sub(a, p)) + r2_norm_sq(r2_sub(d, a)) +
            r2_dot(r2_sub(a, p), r2_sub(d, a)) + r2_dot(r2_sub(a, p), r2_sub(d, a)))
        r2_norm_sq(r2_add(r2_sub(a, p), r2_sub(b, a))) +
            (r2_norm_sq(r2_sub(a, p)) + r2_norm_sq(r2_sub(d, a)) +
            r2_dot(r2_sub(a, p), r2_sub(d, a)) + r2_dot(r2_sub(a, p), r2_sub(d, a))) =
            (r2_norm_sq(r2_add(r2_sub(a, p), r2_sub(b, a))) +
            (r2_norm_sq(r2_sub(a, p)) + r2_norm_sq(r2_sub(d, a))) + r2_dot(r2_sub(a, p), r2_sub(d, a))) +
            r2_dot(r2_sub(a, p), r2_sub(d, a))
        (r2_norm_sq(r2_add(r2_sub(a, p), r2_sub(b, a))) +
            (r2_norm_sq(r2_sub(a, p)) + r2_norm_sq(r2_sub(d, a))) + r2_dot(r2_sub(a, p), r2_sub(d, a))) +
            r2_dot(r2_sub(a, p), r2_sub(d, a)) =
            r2_norm_sq(r2_add(r2_sub(a, p), r2_sub(b, a))) + r2_norm_sq(r2_sub(a, p)) +
            r2_norm_sq(r2_sub(d, a)) +
            r2_dot(r2_sub(a, p), r2_sub(d, a)) + r2_dot(r2_sub(a, p), r2_sub(d, a))
        r2_norm_sq(r2_sub(a, p)) +
            (r2_norm_sq(r2_add(r2_sub(a, p), r2_sub(b, a))) + r2_norm_sq(r2_sub(d, a)) +
            r2_dot(r2_sub(a, p), r2_sub(d, a)) + r2_dot(r2_sub(a, p), r2_sub(d, a))) =
            (r2_norm_sq(r2_sub(a, p)) + (r2_norm_sq(r2_add(r2_sub(a, p), r2_sub(b, a))) + r2_norm_sq(r2_sub(d, a))) +
            r2_dot(r2_sub(a, p), r2_sub(d, a))) + r2_dot(r2_sub(a, p), r2_sub(d, a))
        (r2_norm_sq(r2_sub(a, p)) + (r2_norm_sq(r2_add(r2_sub(a, p), r2_sub(b, a))) + r2_norm_sq(r2_sub(d, a))) +
            r2_dot(r2_sub(a, p), r2_sub(d, a))) + r2_dot(r2_sub(a, p), r2_sub(d, a)) =
            r2_norm_sq(r2_sub(a, p)) + r2_norm_sq(r2_add(r2_sub(a, p), r2_sub(b, a))) +
            r2_norm_sq(r2_sub(d, a)) +
            r2_dot(r2_sub(a, p), r2_sub(d, a)) + r2_dot(r2_sub(a, p), r2_sub(d, a))
        r2_norm_sq(r2_sub(a, p)) + r2_norm_sq(r2_add(r2_sub(a, p), r2_sub(b, a))) +
            r2_norm_sq(r2_sub(d, a)) +
            r2_dot(r2_sub(a, p), r2_sub(d, a)) + r2_dot(r2_sub(a, p), r2_sub(d, a)) =
            r2_norm_sq(r2_add(r2_sub(a, p), r2_sub(b, a))) + r2_norm_sq(r2_sub(a, p)) +
            r2_norm_sq(r2_sub(d, a)) +
            r2_dot(r2_sub(a, p), r2_sub(d, a)) + r2_dot(r2_sub(a, p), r2_sub(d, a))
        r2_norm_sq(r2_sub(a, p)) + r2_norm_sq(r2_sub(c, p)) =
            r2_norm_sq(r2_sub(b, p)) + r2_norm_sq(r2_sub(d, p))
    }
}
