from real import Real
from theorems1000.r2_helpers import real_add_pair_rearrange

numerals Real

// The Brahmagupta-Fibonacci identity (also called the two-square identity
// or Diophantus's identity): the product of two sums of two squares is
// again a sum of two squares,
//     (a^2 + b^2) (c^2 + d^2) = (a c - b d)^2 + (a d + b c)^2.
// It appears in Diophantus's "Arithmetica" (book III, c. 250 AD), was used
// by Brahmagupta (628) for the arithmetic of quadratic forms, and was
// stated in Europe by Fibonacci in his "Liber quadratorum" (1225).  Over
// the integers the identity shows that the set of numbers representable
// as a sum of two squares is closed under multiplication, which is the
// multiplicative half of Fermat's theorem on sums of two squares
// (treated in `number_theory/sum_of_two_squares.ac`).  It is also the
// two-dimensional case of Lagrange's identity; the three- and
// four-dimensional cases are theorem_lagrange_identity.ac and
// theorem_lagrange_four_squares.ac in this directory.
//
// The proof below expands both squares, cancels the four cross terms
// `+-(a c b d + b d a c)` against `+(a d b c + b c a d)` (all four are the
// same monomial `a b c d`), and reassociates the four pure terms into the
// expansion of the left-hand side.

/// The square of a difference.
theorem bf_square_sub(x: Real, y: Real) {
    (x - y) * (x - y) = x * x - x * y - y * x + y * y
} by {
    (x - y) * (x - y) = (x - y) * x - (x - y) * y
    (x - y) * x = x * x - y * x
    (x - y) * y = x * y - y * y
    (x - y) * (x - y) = x * x - y * x - (x * y - y * y)
    y * x = x * y
    x * x - y * x - (x * y - y * y) = x * x - x * y - (x * y - y * y)
    x * x - x * y - (x * y - y * y) = x * x - x * y + -(x * y - y * y)
    -(x * y - y * y) = -(x * y) + y * y
    x * x - x * y + -(x * y - y * y) = x * x - x * y + (-(x * y) + y * y)
    x * x - x * y + (-(x * y) + y * y) = x * x - x * y - x * y + y * y
    x * x - x * y - x * y + y * y = x * x - x * y - y * x + y * y
}

/// The square of a two-term sum.
theorem bf_square_add(x: Real, y: Real) {
    (x + y) * (x + y) = x * x + y * x + (x * y + y * y)
} by {
    (x + y) * (x + y) = (x + y) * x + (x + y) * y
    (x + y) * x = x * x + y * x
    (x + y) * y = x * y + y * y
    (x + y) * (x + y) = x * x + y * x + (x * y + y * y)
}

/// A difference of three terms expands to a sum of negations.
theorem bf_minus_to_add(x: Real, m: Real, n: Real, y: Real) {
    x - m - n + y = x + -m + -n + y
}

/// A sum of negations regroups as a difference of sums.
theorem bf_add_to_minus(x: Real, m: Real, n: Real, y: Real) {
    x + -m + -n + y = (x - m) + (y - n)
} by {
    x + -m + -n + y = (x + -m) + (-n + y)
    (x + -m) + (-n + y) = (x - m) + (y - n)
}

/// Cancelling a term and its negation: `x - m + (m + z) = x + z`.
theorem bf_cancel_sub_add(x: Real, m: Real, z: Real) {
    x - m + (m + z) = x + z
} by {
    x - m = x + -m
    x + -m + (m + z) = x + (-m + (m + z))
    -m + (m + z) = (-m + m) + z
    -m + m = Real.0
    (-m + m) + z = Real.0 + z
    Real.0 + z = z
    x + (-m + (m + z)) = x + z
}

/// Cancelling a swapped term and its negation: `x - m + (z + m) = x + z`.
theorem bf_cancel_swap_add(x: Real, m: Real, z: Real) {
    x - m + (z + m) = x + z
} by {
    z + m = m + z
    x - m + (z + m) = x - m + (m + z)
    bf_cancel_sub_add(x, m, z)
    x - m + (m + z) = x + z
}

/// Opposite middle terms cancel in a sum of two expanded squares:
/// `(x - m - n + y) + (z + m + n + w) = x + y + z + w`.
theorem bf_cancel_middle(x: Real, y: Real, z: Real, w: Real, m: Real, n: Real) {
    (x - m - n + y) + (z + m + n + w) = x + y + z + w
} by {
    bf_minus_to_add(x, m, n, y)
    x - m - n + y = x + -m + -n + y
    bf_add_to_minus(x, m, n, y)
    x + -m + -n + y = (x - m) + (y - n)
    z + m + n + w = (z + m) + (n + w)
    real_add_pair_rearrange(x - m, y - n, z + m, n + w)
    ((x - m) + (y - n)) + ((z + m) + (n + w)) =
        ((x - m) + (z + m)) + ((y - n) + (n + w))
    bf_cancel_swap_add(x, m, z)
    x - m + (z + m) = x + z
    bf_cancel_sub_add(y, n, w)
    y - n + (n + w) = y + w
    ((x - m) + (z + m)) + ((y - n) + (n + w)) = (x + z) + (y + w)
}

/// The four pure terms of the two expanded squares reassemble into the
/// expansion of the product.
theorem bf_pure_reassemble(a: Real, b: Real, c: Real, d: Real) {
    a * c * a * c + b * d * b * d + a * d * a * d + b * c * b * c =
        a * a * c * c + a * a * d * d + b * b * c * c + b * b * d * d
} by {
    a * c * a * c = a * a * c * c
    b * d * b * d = b * b * d * d
    a * d * a * d = a * a * d * d
    b * c * b * c = b * b * c * c
    a * a * c * c + b * b * d * d + a * a * d * d + b * b * c * c =
        (a * a * c * c + b * b * d * d) + (a * a * d * d + b * b * c * c)
    real_add_pair_rearrange(a * a * c * c, b * b * d * d, a * a * d * d, b * b * c * c)
    (a * a * c * c + b * b * d * d) + (a * a * d * d + b * b * c * c) =
        (a * a * c * c + a * a * d * d) + (b * b * d * d + b * b * c * c)
    b * b * d * d + b * b * c * c = b * b * c * c + b * b * d * d
    (a * a * c * c + a * a * d * d) + (b * b * d * d + b * b * c * c) =
        (a * a * c * c + a * a * d * d) + (b * b * c * c + b * b * d * d)
    (a * a * c * c + a * a * d * d) + (b * b * c * c + b * b * d * d) =
        a * a * c * c + a * a * d * d + b * b * c * c + b * b * d * d
}

/// The left-hand side expands to four pure terms.
theorem bf_lhs_expand(a: Real, b: Real, c: Real, d: Real) {
    (a * a + b * b) * (c * c + d * d) =
        a * a * c * c + a * a * d * d + b * b * c * c + b * b * d * d
} by {
    (a * a + b * b) * (c * c + d * d) = (a * a + b * b) * (c * c) + (a * a + b * b) * (d * d)
    (a * a + b * b) * (c * c) = a * a * (c * c) + b * b * (c * c)
    (a * a + b * b) * (d * d) = a * a * (d * d) + b * b * (d * d)
    (a * a + b * b) * (c * c + d * d) =
        a * a * (c * c) + b * b * (c * c) + (a * a * (d * d) + b * b * (d * d))
    a * a * (c * c) + b * b * (c * c) + (a * a * (d * d) + b * b * (d * d)) =
        a * a * (c * c) + (b * b * (c * c) + a * a * (d * d)) + b * b * (d * d)
    b * b * (c * c) + a * a * (d * d) = a * a * (d * d) + b * b * (c * c)
    a * a * (c * c) + (b * b * (c * c) + a * a * (d * d)) + b * b * (d * d) =
        a * a * (c * c) + (a * a * (d * d) + b * b * (c * c)) + b * b * (d * d)
    a * a * (c * c) + (a * a * (d * d) + b * b * (c * c)) + b * b * (d * d) =
        a * a * (c * c) + a * a * (d * d) + b * b * (c * c) + b * b * (d * d)
    a * a * (c * c) + a * a * (d * d) + b * b * (c * c) + b * b * (d * d) =
        a * a * c * c + a * a * d * d + b * b * c * c + b * b * d * d
}

/// The Brahmagupta-Fibonacci identity: the product of two sums of two
/// squares is a sum of two squares.
theorem theorems1000_brahmagupta_fibonacci(a: Real, b: Real, c: Real, d: Real) {
    (a * a + b * b) * (c * c + d * d) =
        (a * c - b * d) * (a * c - b * d) + (a * d + b * c) * (a * d + b * c)
} by {
    bf_square_sub(a * c, b * d)
    (a * c - b * d) * (a * c - b * d) =
        a * c * a * c - a * c * b * d - b * d * a * c + b * d * b * d
    bf_square_add(a * d, b * c)
    (a * d + b * c) * (a * d + b * c) =
        a * d * a * d + b * c * a * d + (a * d * b * c + b * c * b * c)
    a * d * b * c = a * c * b * d
    b * c * a * d = b * d * a * c
    bf_cancel_middle(a * c * a * c, b * d * b * d, a * d * a * d, b * c * b * c,
        a * c * b * d, b * d * a * c)
    a * c * a * c - a * c * b * d - b * d * a * c + b * d * b * d +
        (a * d * a * d + a * c * b * d + b * d * a * c + b * c * b * c) =
        a * c * a * c + b * d * b * d + a * d * a * d + b * c * b * c
    bf_pure_reassemble(a, b, c, d)
    a * c * a * c + b * d * b * d + a * d * a * d + b * c * b * c =
        a * a * c * c + a * a * d * d + b * b * c * c + b * b * d * d
    bf_lhs_expand(a, b, c, d)
    (a * a + b * b) * (c * c + d * d) =
        a * a * c * c + a * a * d * d + b * b * c * c + b * b * d * d
}

// Over the natural numbers the identity gives the closure of the set of
// sums of two squares under multiplication: if `m = a^2 + b^2` and
// `n = c^2 + d^2` then `m n` is again a sum of two squares.  A direct
// natural-number proof must split on whether `a d >= b c` (in which case
// `m n = (a c + b d)^2 + (a d - b c)^2`) or `a d < b c` (in which case
// `m n = (a c + b d)^2 + (b c - a d)^2`), because natural-number
// subtraction is truncated; the identity itself is a statement about the
// integers, so it is recorded here over the reals.
//
// theorem theorems1000_two_squares_closed(m: Nat, n: Nat) {
//     exists(a: Nat, b: Nat) { m = a * a + b * b } and
//     exists(c: Nat, d: Nat) { n = c * c + d * d }
//         implies exists(e: Nat, f: Nat) { m * n = e * e + f * f }
// }
