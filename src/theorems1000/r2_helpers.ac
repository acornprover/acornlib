from pair import Pair, pair_ext
from real import Real, mul_left_cancel, one_half_plus_one_half
from top100 import r2_add, r2_neg, r2_sub, r2_dot, r2_norm_sq, r2_orthogonal

// Shared helpers for the theorems1000 geometry theorems.
// Points are represented as pairs of reals, with the componentwise
// operations `r2_add`, `r2_neg`, `r2_sub`, the dot product `r2_dot`, and
// the squared norm `r2_norm_sq` (from top100's interface).  The midpoint
// hypothesis "`m` is the midpoint of `b` and `c`" is written
// `r2_add(b, c) = r2_add(m, m)`, i.e. `b + c = 2m`.

// ---------------------------------------------------------------------------
// Scalar (real) helpers.
// ---------------------------------------------------------------------------

/// The square of a sum, with the two cross terms written separately.
theorem real_square_add(x: Real, y: Real) {
    (x + y) * (x + y) = x * x + x * y + (x * y + y * y)
} by {
    (x + y) * (x + y) = (x + y) * x + (x + y) * y
    (x + y) * x = x * x + y * x
    (x + y) * y = x * y + y * y
    (x + y) * (x + y) = (x * x + y * x) + (x * y + y * y)
    y * x = x * y
    (x * x + y * x) + (x * y + y * y) = x * x + x * y + (x * y + y * y)
}

/// Move the repeated middle term together in a four-term sum.
theorem real_add_repeated_middle(a: Real, b: Real, c: Real) {
    a + b + (b + c) = a + (b + b) + c
} by {
    a + b + (b + c) = (a + b) + (b + c)
    (a + b) + (b + c) = a + (b + (b + c))
    b + (b + c) = (b + b) + c
    a + (b + (b + c)) = a + ((b + b) + c)
    a + ((b + b) + c) = a + (b + b) + c
}

/// Repeated terms may be paired in either order.
theorem real_double_pair_rearrange(a: Real, b: Real) {
    (a + a) + (b + b) = (a + b) + (a + b)
} by {
    (a + a) + (b + b) = a + (a + (b + b))
    a + (b + (a + b)) = (a + b) + (a + b)
    a + (a + (b + b)) = a + (b + (a + b))
}

/// Move the last term of a four-term sum next to the first.
theorem real_add_four_last_next_to_first(a: Real, b: Real, c: Real, d: Real) {
    a + b + c + d = (a + d) + (b + c)
} by {
    a + b + c + d = ((a + b) + c) + d
    ((a + b) + c) + d = (a + b) + (c + d)
    (a + b) + (c + d) = a + (b + (c + d))
    b + (c + d) = (b + c) + d
    a + ((b + c) + d) = a + (d + (b + c))
    a + (d + (b + c)) = (a + d) + (b + c)
}

/// Pair the first, fourth, third, and sixth terms of a six-term sum.
theorem real_add_six_pair_rearrange(a: Real, b: Real, c: Real, d: Real, e: Real, f: Real) {
    (a + b + c) + (d + e + f) = (a + d) + (c + f) + (b + e)
} by {
    (a + b + c) + (d + e + f) = ((a + b + c) + (d + e)) + f
    ((a + b + c) + (d + e)) + f = (((a + b + c) + d) + e) + f
    real_add_four_last_next_to_first(a, b, c, d)
    a + b + c + d = (a + d) + (b + c)
    (((a + b + c) + d) + e) + f = (((a + d) + (b + c)) + e) + f
    ((a + d) + (b + c)) + e + f = ((a + d) + ((b + c) + e)) + f
    (b + c) + e = c + (b + e)
    ((a + d) + ((b + c) + e)) + f = ((a + d) + (c + (b + e))) + f
    ((a + d) + (c + (b + e))) + f = (a + d) + ((c + (b + e)) + f)
    (c + (b + e)) + f = (c + f) + (b + e)
    (a + d) + ((c + f) + (b + e)) = (a + d) + (c + f) + (b + e)
}

/// Pairing two sums can be regrouped: (a + b) + (c + d) = (a + c) + (b + d).
theorem real_add_pair_rearrange(a: Real, b: Real, c: Real, d: Real) {
    (a + b) + (c + d) = (a + c) + (b + d)
} by {
    (a + b) + (c + d) = a + b + c + d
    real_add_four_last_next_to_first(a, b, c, d)
    a + b + c + d = (a + d) + (b + c)
    (a + d) + (b + c) = a + (d + (b + c))
    a + (d + (b + c)) = a + (b + (d + c))
    d + c = c + d
    b + (d + c) = b + (c + d)
    a + (b + (d + c)) = a + (b + (c + d))
    b + (c + d) = (b + c) + d
    a + (b + (c + d)) = a + ((b + c) + d)
    (b + c) + d = (c + b) + d
    c + b = b + c
    a + ((b + c) + d) = a + ((c + b) + d)
    (c + b) + d = c + (b + d)
    a + ((c + b) + d) = a + (c + (b + d))
    a + (c + (b + d)) = (a + c) + (b + d)
}

/// Subtraction through an intermediate real number decomposes a difference.
theorem real_sub_through(x: Real, y: Real, z: Real) {
    (y - x) + (z - y) = z - x
} by {
    y - x = y + -x
    z - y = z + -y
    z - x = z + -x
    (y - x) + (z - y) = (y + -x) + (z + -y)
    (y + -x) + (z + -y) = y + -x + z + -y
    real_add_four_last_next_to_first(y, -x, z, -y)
    y + -x + z + -y = (y + -y) + (-x + z)
    y + -y = Real.0
    (y + -y) + (-x + z) = Real.0 + (-x + z)
    Real.0 + (-x + z) = -x + z
    -x + z = z + -x
    (y - x) + (z - y) = z - x
}

/// Reversing a real difference negates it.
theorem real_sub_reverse_neg(x: Real, y: Real) {
    x - y = -(y - x)
} by {
    x - y = x + -y
    y - x = y + -x
    -(y - x) = -(y + -x)
    -(y + -x) = -y + --x
    --x = x
    -y + x = x + -y
}

/// Distribution of subtraction over a sum of differences:
/// (a - b) + (c - d) = (a + c) - (b + d).
theorem real_sub_add_distrib(a: Real, b: Real, c: Real, d: Real) {
    (a - b) + (c - d) = (a + c) - (b + d)
} by {
    a - b = a + -b
    c - d = c + -d
    (a - b) + (c - d) = (a + -b) + (c + -d)
    (a + -b) + (c + -d) = a + -b + c + -d
    real_add_four_last_next_to_first(a, -b, c, -d)
    a + -b + c + -d = (a + c) + (-b + -d)
    -b + -d = -(b + d)
    (a + c) + (-b + -d) = (a + c) + -(b + d)
    (a + c) - (b + d) = (a + c) + -(b + d)
}

/// Cancelling a doubled middle term in a sum of differences:
/// (a + c) - (c + b) = a - b.
theorem real_add_sub_cancel_middle(a: Real, b: Real, c: Real) {
    (a + c) - (c + b) = a - b
} by {
    a + c = c + a
    c + b = b + c
    (a + c) - (c + b) = (c + a) - (b + c)
    (c + a) - (b + c) = c + a + -(b + c)
    -(b + c) = -b + -c
    c + a + (-b + -c) = c + a + -b + -c
    real_add_four_last_next_to_first(c, a, -b, -c)
    c + a + -b + -c = (c + -c) + (a + -b)
    c + -c = Real.0
    (c + -c) + (a + -b) = Real.0 + (a + -b)
    Real.0 + (a + -b) = a + -b
    a - b = a + -b
}

/// The square of a negation is the square.
theorem real_neg_sq(x: Real) {
    (-x) * (-x) = x * x
} by {
    (-x) * (-x) = -(x * (-x))
    x * (-x) = -(x * x)
    -(-(x * x)) = x * x
}

/// A doubled term and its negated double cancel: d + d + (-d + -d) = 0.
theorem real_double_cancel(d: Real) {
    d + d + (-d + -d) = Real.0
} by {
    d + d + (-d + -d) = d + d + -d + -d
    d + d + -d + -d = (d + -d) + (d + -d)
    d + -d = Real.0
    (d + -d) + (d + -d) = Real.0 + Real.0
    Real.0 + Real.0 = Real.0
}

/// Doubling a real is the sum of it with itself.
theorem real_double_eq_add_self(a: Real) {
    (Real.1 + Real.1) * a = a + a
}

/// From `b + c = 2m`, the real difference `c - m` is the negation of `b - m`.
theorem real_neg_sub_swap(b: Real, c: Real, m: Real) {
    b + c = (Real.1 + Real.1) * m implies c - m = -(b - m)
} by {
    if b + c = (Real.1 + Real.1) * m {
        real_double_eq_add_self(m)
        (Real.1 + Real.1) * m = m + m
        b + c = m + m
        b + c + -m = m + m + -m
        m + m + -m = m + (m + -m)
        m + -m = Real.0
        m + (m + -m) = m + Real.0
        m + Real.0 = m
        b + c + -m = m
        b + (c + -m) = m
        b + (c + -m) + -b = m + -b
        b + (c + -m) + -b = (b + (c + -m)) + -b
        (b + (c + -m)) + -b = (b + (c + -m)) + (-b + Real.0)
        real_add_pair_rearrange(b, c + -m, -b, Real.0)
        (b + (c + -m)) + (-b + Real.0) = (b + -b) + ((c + -m) + Real.0)
        b + -b = Real.0
        (c + -m) + Real.0 = c + -m
        (b + -b) + ((c + -m) + Real.0) = Real.0 + (c + -m)
        Real.0 + (c + -m) = c + -m
        b + (c + -m) + -b = c + -m
        c + -m = m + -b
        m + -b = -(b - m)
        b - m = b + -m
        -(b - m) = -(b + -m)
        -(b + -m) = -b + m
        -b + m = m + -b
        c - m = c + -m
        c - m = -(b - m)
    }
}

/// From `a + b = 2m`, the real difference `m - b` equals `a - m`.
theorem real_mid_sub_swap(a: Real, b: Real, m: Real) {
    a + b = (Real.1 + Real.1) * m implies m - b = a - m
} by {
    if a + b = (Real.1 + Real.1) * m {
        real_sub_reverse_neg(m, b)
        m - b = -(b - m)
        b + a = a + b
        b + a = (Real.1 + Real.1) * m
        real_neg_sub_swap(b, a, m)
        a - m = -(b - m)
        a - m = m - b
    }
}

// ---------------------------------------------------------------------------
// Point (pair) helpers.
// ---------------------------------------------------------------------------

/// Negation may be moved out of the left side of the dot product.
theorem r2_dot_neg_left(a: Pair[Real, Real], b: Pair[Real, Real]) {
    r2_dot(r2_neg(a), b) = -r2_dot(a, b)
} by {
    r2_neg(a).first = -a.first
    r2_neg(a).second = -a.second
    r2_dot(r2_neg(a), b) = r2_neg(a).first * b.first + r2_neg(a).second * b.second
    r2_dot(r2_neg(a), b) = (-a.first) * b.first + (-a.second) * b.second
    (-a.first) * b.first = -(a.first * b.first)
    (-a.second) * b.second = -(a.second * b.second)
    r2_dot(a, b) = a.first * b.first + a.second * b.second
    -(a.first * b.first + a.second * b.second) = -(a.first * b.first) + -(a.second * b.second)
    r2_dot(r2_neg(a), b) = -r2_dot(a, b)
}

/// Negation may be moved out of the right side of the dot product.
theorem r2_dot_neg_right(a: Pair[Real, Real], b: Pair[Real, Real]) {
    r2_dot(a, r2_neg(b)) = -r2_dot(a, b)
} by {
    r2_neg(b).first = -b.first
    r2_neg(b).second = -b.second
    r2_dot(a, r2_neg(b)) = a.first * r2_neg(b).first + a.second * r2_neg(b).second
    r2_dot(a, r2_neg(b)) = a.first * (-b.first) + a.second * (-b.second)
    a.first * (-b.first) = -(a.first * b.first)
    a.second * (-b.second) = -(a.second * b.second)
    r2_dot(a, b) = a.first * b.first + a.second * b.second
    -(a.first * b.first + a.second * b.second) = -(a.first * b.first) + -(a.second * b.second)
    r2_dot(a, r2_neg(b)) = -r2_dot(a, b)
}

/// Subtracting a vector equals adding its negation.
theorem r2_sub_eq_add_neg(a: Pair[Real, Real], b: Pair[Real, Real]) {
    r2_sub(a, b) = r2_add(a, r2_neg(b))
} by {
    let lhs = r2_sub(a, b)
    let rhs = r2_add(a, r2_neg(b))
    lhs.first = a.first - b.first
    lhs.second = a.second - b.second
    rhs.first = a.first + r2_neg(b).first
    r2_neg(b).first = -b.first
    rhs.first = a.first + -b.first
    rhs.second = a.second + r2_neg(b).second
    r2_neg(b).second = -b.second
    rhs.second = a.second + -b.second
    lhs.first = rhs.first
    lhs.second = rhs.second
    pair_ext(lhs, rhs)
}

/// A point added to its negation is the origin.
theorem r2_add_neg_right(a: Pair[Real, Real]) {
    r2_add(a, r2_neg(a)) = Pair.new(Real.0, Real.0)
} by {
    let lhs = r2_add(a, r2_neg(a))
    lhs.first = a.first + r2_neg(a).first
    r2_neg(a).first = -a.first
    lhs.first = a.first + -a.first
    lhs.second = a.second + r2_neg(a).second
    r2_neg(a).second = -a.second
    lhs.second = a.second + -a.second
    a.first + -a.first = Real.0
    a.second + -a.second = Real.0
    lhs.first = Real.0
    lhs.second = Real.0
    pair_ext(lhs, Pair.new(Real.0, Real.0))
}

/// Reversing the order in a coordinate-pair subtraction negates the vector.
theorem r2_sub_reverse_neg(a: Pair[Real, Real], b: Pair[Real, Real]) {
    r2_sub(a, b) = r2_neg(r2_sub(b, a))
} by {
    let lhs = r2_sub(a, b)
    let rhs = r2_neg(r2_sub(b, a))
    lhs.first = a.first - b.first
    lhs.second = a.second - b.second
    rhs.first = -r2_sub(b, a).first
    r2_sub(b, a).first = b.first - a.first
    rhs.first = -(b.first - a.first)
    real_sub_reverse_neg(a.first, b.first)
    a.first - b.first = -(b.first - a.first)
    rhs.first = lhs.first
    rhs.second = -r2_sub(b, a).second
    r2_sub(b, a).second = b.second - a.second
    rhs.second = -(b.second - a.second)
    real_sub_reverse_neg(a.second, b.second)
    a.second - b.second = -(b.second - a.second)
    rhs.second = lhs.second
    pair_ext(lhs, rhs)
}

/// Orthogonality is unchanged by negating the left vector.
theorem r2_orthogonal_neg_left(a: Pair[Real, Real], b: Pair[Real, Real]) {
    r2_orthogonal(r2_neg(a), b) implies r2_orthogonal(a, b)
} by {
    if r2_orthogonal(r2_neg(a), b) {
        r2_dot(r2_neg(a), b) = Real.0
        r2_dot_neg_left(a, b)
        r2_dot(r2_neg(a), b) = -r2_dot(a, b)
        -r2_dot(a, b) = Real.0
        r2_dot(a, b) = Real.0
        r2_orthogonal(a, b)
    }
}

/// Rearrangement of the six terms that occur in the two-coordinate expansion.
theorem real_pythagoras_rearrange(xx: Real, yy: Real, uu: Real, vv: Real, xu: Real, yv: Real) {
    (xx + xu + (xu + uu)) + (yy + yv + (yv + vv)) =
    (xx + yy) + (uu + vv) + (xu + yv) + (xu + yv)
} by {
    real_add_repeated_middle(xx, xu, uu)
    real_add_repeated_middle(yy, yv, vv)
    xx + xu + (xu + uu) = xx + (xu + xu) + uu
    yy + yv + (yv + vv) = yy + (yv + yv) + vv
    real_double_pair_rearrange(xu, yv)
    (xu + xu) + (yv + yv) = (xu + yv) + (xu + yv)
    real_add_six_pair_rearrange(xx, xu + xu, uu, yy, yv + yv, vv)
    (xx + (xu + xu) + uu) + (yy + (yv + yv) + vv) =
        (xx + yy) + (uu + vv) + ((xu + xu) + (yv + yv))
    (xx + yy) + (uu + vv) + ((xu + xu) + (yv + yv)) =
        (xx + yy) + (uu + vv) + (xu + yv) + (xu + yv)
}

/// The squared norm of a sum expands by the dot product.
theorem r2_norm_sq_add_expansion(a: Pair[Real, Real], b: Pair[Real, Real]) {
    r2_norm_sq(r2_add(a, b)) = r2_norm_sq(a) + r2_norm_sq(b) + r2_dot(a, b) + r2_dot(a, b)
} by {
    r2_norm_sq(r2_add(a, b)) = r2_dot(r2_add(a, b), r2_add(a, b))
    r2_dot(r2_add(a, b), r2_add(a, b)) =
        r2_add(a, b).first * r2_add(a, b).first + r2_add(a, b).second * r2_add(a, b).second
    r2_add(a, b).first = a.first + b.first
    r2_add(a, b).second = a.second + b.second
    r2_add(a, b).first * r2_add(a, b).first + r2_add(a, b).second * r2_add(a, b).second =
        (a.first + b.first) * r2_add(a, b).first + (a.second + b.second) * r2_add(a, b).second
    (a.first + b.first) * r2_add(a, b).first + (a.second + b.second) * r2_add(a, b).second =
        (a.first + b.first) * (a.first + b.first) + (a.second + b.second) * (a.second + b.second)
    real_square_add(a.first, b.first)
    real_square_add(a.second, b.second)
    real_pythagoras_rearrange(
        a.first * a.first,
        a.second * a.second,
        b.first * b.first,
        b.second * b.second,
        a.first * b.first,
        a.second * b.second)
    (a.first + b.first) * (a.first + b.first) +
        (a.second + b.second) * (a.second + b.second) =
        (a.first * a.first + a.second * a.second) +
        (b.first * b.first + b.second * b.second) +
        (a.first * b.first + a.second * b.second) +
        (a.first * b.first + a.second * b.second)
    r2_norm_sq(a) = a.first * a.first + a.second * a.second
    r2_norm_sq(b) = b.first * b.first + b.second * b.second
    r2_dot(a, b) = a.first * b.first + a.second * b.second
}

/// The squared norm of a difference expands by the dot product.
theorem r2_norm_sq_sub_expansion(a: Pair[Real, Real], b: Pair[Real, Real]) {
    r2_norm_sq(r2_sub(a, b)) = r2_norm_sq(a) + r2_norm_sq(b) - r2_dot(a, b) - r2_dot(a, b)
} by {
    r2_sub_eq_add_neg(a, b)
    r2_sub(a, b) = r2_add(a, r2_neg(b))
    r2_norm_sq(r2_sub(a, b)) = r2_norm_sq(r2_add(a, r2_neg(b)))
    r2_norm_sq_add_expansion(a, r2_neg(b))
    r2_norm_sq(r2_add(a, r2_neg(b))) =
        r2_norm_sq(a) + r2_norm_sq(r2_neg(b)) + r2_dot(a, r2_neg(b)) + r2_dot(a, r2_neg(b))
    r2_norm_sq(r2_neg(b)) = r2_dot(r2_neg(b), r2_neg(b))
    r2_dot(r2_neg(b), r2_neg(b)) =
        r2_neg(b).first * r2_neg(b).first + r2_neg(b).second * r2_neg(b).second
    r2_neg(b).first = -b.first
    r2_neg(b).second = -b.second
    r2_neg(b).first * r2_neg(b).first + r2_neg(b).second * r2_neg(b).second =
        (-b.first) * (-b.first) + (-b.second) * (-b.second)
    real_neg_sq(b.first)
    real_neg_sq(b.second)
    (-b.first) * (-b.first) + (-b.second) * (-b.second) =
        b.first * b.first + b.second * b.second
    r2_norm_sq(b) = r2_dot(b, b)
    r2_dot(b, b) = b.first * b.first + b.second * b.second
    r2_norm_sq(r2_neg(b)) = r2_norm_sq(b)
    r2_dot_neg_right(a, b)
    r2_dot(a, r2_neg(b)) = -r2_dot(a, b)
    r2_norm_sq(a) + r2_norm_sq(r2_neg(b)) + r2_dot(a, r2_neg(b)) + r2_dot(a, r2_neg(b)) =
        r2_norm_sq(a) + r2_norm_sq(b) + -r2_dot(a, b) + -r2_dot(a, b)
    r2_norm_sq(a) + r2_norm_sq(b) + -r2_dot(a, b) + -r2_dot(a, b) =
        r2_norm_sq(a) + r2_norm_sq(b) - r2_dot(a, b) - r2_dot(a, b)
    r2_norm_sq(r2_sub(a, b)) = r2_norm_sq(a) + r2_norm_sq(b) - r2_dot(a, b) - r2_dot(a, b)
}

/// Pythagoras for orthogonal vectors in the real coordinate plane.
theorem r2_pythagoras_vectors(a: Pair[Real, Real], b: Pair[Real, Real]) {
    r2_orthogonal(a, b) implies
    r2_norm_sq(r2_add(a, b)) = r2_norm_sq(a) + r2_norm_sq(b)
} by {
    if r2_orthogonal(a, b) {
        r2_dot(a, b) = Real.0
        r2_norm_sq_add_expansion(a, b)
        r2_norm_sq(r2_add(a, b)) = r2_norm_sq(a) + r2_norm_sq(b) + r2_dot(a, b) + r2_dot(a, b)
        r2_norm_sq(a) + r2_norm_sq(b) + r2_dot(a, b) + r2_dot(a, b) =
            r2_norm_sq(a) + r2_norm_sq(b) + r2_dot(a, b) + Real.0
        r2_norm_sq(a) + r2_norm_sq(b) + r2_dot(a, b) + Real.0 =
            r2_norm_sq(a) + r2_norm_sq(b) + Real.0 + Real.0
        r2_norm_sq(a) + r2_norm_sq(b) + Real.0 + Real.0 = r2_norm_sq(a) + r2_norm_sq(b)
        r2_norm_sq(r2_add(a, b)) = r2_norm_sq(a) + r2_norm_sq(b)
    }
}

/// Subtraction through an intermediate point decomposes a displacement.
theorem r2_sub_through(a: Pair[Real, Real], b: Pair[Real, Real], c: Pair[Real, Real]) {
    r2_sub(c, a) = r2_add(r2_sub(b, a), r2_sub(c, b))
} by {
    let lhs = r2_sub(c, a)
    let rhs = r2_add(r2_sub(b, a), r2_sub(c, b))
    lhs.first = c.first - a.first
    lhs.second = c.second - a.second
    rhs.first = r2_sub(b, a).first + r2_sub(c, b).first
    r2_sub(b, a).first = b.first - a.first
    r2_sub(c, b).first = c.first - b.first
    rhs.first = (b.first - a.first) + (c.first - b.first)
    real_sub_through(a.first, b.first, c.first)
    (b.first - a.first) + (c.first - b.first) = c.first - a.first
    rhs.first = lhs.first
    rhs.second = r2_sub(b, a).second + r2_sub(c, b).second
    r2_sub(b, a).second = b.second - a.second
    r2_sub(c, b).second = c.second - b.second
    rhs.second = (b.second - a.second) + (c.second - b.second)
    real_sub_through(a.second, b.second, c.second)
    (b.second - a.second) + (c.second - b.second) = c.second - a.second
    rhs.second = lhs.second
    pair_ext(lhs, rhs)
}

/// The dot product distributes over addition on the left.
theorem r2_dot_add_left(a: Pair[Real, Real], b: Pair[Real, Real], c: Pair[Real, Real]) {
    r2_dot(r2_add(a, b), c) = r2_dot(a, c) + r2_dot(b, c)
} by {
    r2_dot(r2_add(a, b), c) = r2_add(a, b).first * c.first + r2_add(a, b).second * c.second
    r2_add(a, b).first = a.first + b.first
    r2_add(a, b).second = a.second + b.second
    r2_dot(r2_add(a, b), c) = (a.first + b.first) * c.first + (a.second + b.second) * c.second
    (a.first + b.first) * c.first = a.first * c.first + b.first * c.first
    (a.second + b.second) * c.second = a.second * c.second + b.second * c.second
    (a.first + b.first) * c.first + (a.second + b.second) * c.second =
        (a.first * c.first + b.first * c.first) + (a.second * c.second + b.second * c.second)
    real_add_pair_rearrange(a.first * c.first, b.first * c.first, a.second * c.second, b.second * c.second)
    (a.first * c.first + b.first * c.first) + (a.second * c.second + b.second * c.second) =
        (a.first * c.first + a.second * c.second) + (b.first * c.first + b.second * c.second)
    r2_dot(a, c) = a.first * c.first + a.second * c.second
    r2_dot(b, c) = b.first * c.first + b.second * c.second
}

/// The dot product distributes over addition on the right.
theorem r2_dot_add_right(a: Pair[Real, Real], b: Pair[Real, Real], c: Pair[Real, Real]) {
    r2_dot(a, r2_add(b, c)) = r2_dot(a, b) + r2_dot(a, c)
} by {
    r2_dot(a, r2_add(b, c)) = a.first * r2_add(b, c).first + a.second * r2_add(b, c).second
    r2_add(b, c).first = b.first + c.first
    r2_add(b, c).second = b.second + c.second
    r2_dot(a, r2_add(b, c)) = a.first * (b.first + c.first) + a.second * (b.second + c.second)
    a.first * (b.first + c.first) = a.first * b.first + a.first * c.first
    a.second * (b.second + c.second) = a.second * b.second + a.second * c.second
    a.first * (b.first + c.first) + a.second * (b.second + c.second) =
        (a.first * b.first + a.first * c.first) + (a.second * b.second + a.second * c.second)
    real_add_pair_rearrange(a.first * b.first, a.first * c.first, a.second * b.second, a.second * c.second)
    (a.first * b.first + a.first * c.first) + (a.second * b.second + a.second * c.second) =
        (a.first * b.first + a.second * b.second) + (a.first * c.first + a.second * c.second)
    r2_dot(a, b) = a.first * b.first + a.second * b.second
    r2_dot(a, c) = a.first * c.first + a.second * c.second
}

/// The dot product distributes over subtraction on the left.
theorem r2_dot_sub_left(a: Pair[Real, Real], b: Pair[Real, Real], c: Pair[Real, Real]) {
    r2_dot(r2_sub(a, b), c) = r2_dot(a, c) - r2_dot(b, c)
} by {
    r2_dot(r2_sub(a, b), c) = r2_sub(a, b).first * c.first + r2_sub(a, b).second * c.second
    r2_sub(a, b).first = a.first - b.first
    r2_sub(a, b).second = a.second - b.second
    r2_dot(r2_sub(a, b), c) = (a.first - b.first) * c.first + (a.second - b.second) * c.second
    (a.first - b.first) * c.first = a.first * c.first - b.first * c.first
    (a.second - b.second) * c.second = a.second * c.second - b.second * c.second
    r2_dot(a, c) = a.first * c.first + a.second * c.second
    r2_dot(b, c) = b.first * c.first + b.second * c.second
    real_sub_add_distrib(a.first * c.first, b.first * c.first, a.second * c.second, b.second * c.second)
    (a.first * c.first - b.first * c.first) + (a.second * c.second - b.second * c.second) =
        (a.first * c.first + a.second * c.second) - (b.first * c.first + b.second * c.second)
}

/// The dot product distributes over subtraction on the right.
theorem r2_dot_sub_right(a: Pair[Real, Real], b: Pair[Real, Real], c: Pair[Real, Real]) {
    r2_dot(a, r2_sub(b, c)) = r2_dot(a, b) - r2_dot(a, c)
} by {
    r2_dot(a, r2_sub(b, c)) = a.first * r2_sub(b, c).first + a.second * r2_sub(b, c).second
    r2_sub(b, c).first = b.first - c.first
    r2_sub(b, c).second = b.second - c.second
    r2_dot(a, r2_sub(b, c)) = a.first * (b.first - c.first) + a.second * (b.second - c.second)
    a.first * (b.first - c.first) = a.first * b.first - a.first * c.first
    a.second * (b.second - c.second) = a.second * b.second - a.second * c.second
    r2_dot(a, b) = a.first * b.first + a.second * b.second
    r2_dot(a, c) = a.first * c.first + a.second * c.second
    real_sub_add_distrib(a.first * b.first, a.first * c.first, a.second * b.second, a.second * c.second)
    (a.first * b.first - a.first * c.first) + (a.second * b.second - a.second * c.second) =
        (a.first * b.first + a.second * b.second) - (a.first * c.first + a.second * c.second)
}

/// The dot product of a difference with a sum is the difference of the norms.
theorem r2_dot_sub_add(u: Pair[Real, Real], v: Pair[Real, Real]) {
    r2_dot(r2_sub(u, v), r2_add(u, v)) = r2_norm_sq(u) - r2_norm_sq(v)
} by {
    r2_dot_sub_left(u, v, r2_add(u, v))
    r2_dot(r2_sub(u, v), r2_add(u, v)) = r2_dot(u, r2_add(u, v)) - r2_dot(v, r2_add(u, v))
    r2_dot_add_right(u, u, v)
    r2_dot(u, r2_add(u, v)) = r2_dot(u, u) + r2_dot(u, v)
    r2_dot_add_right(v, u, v)
    r2_dot(v, r2_add(u, v)) = r2_dot(v, u) + r2_dot(v, v)
    r2_dot(u, u) = r2_norm_sq(u)
    r2_dot(v, v) = r2_norm_sq(v)
    r2_dot(v, u) = r2_dot(u, v)
    real_add_sub_cancel_middle(r2_norm_sq(u), r2_norm_sq(v), r2_dot(u, v))
    r2_norm_sq(u) + r2_dot(u, v) - (r2_dot(u, v) + r2_norm_sq(v)) =
        r2_norm_sq(u) - r2_norm_sq(v)
}

/// The parallelogram law: the sum of the squared norms of the diagonals of a
/// parallelogram is twice the sum of the squared norms of the sides.
theorem r2_parallelogram_law(u: Pair[Real, Real], v: Pair[Real, Real]) {
    r2_norm_sq(r2_add(u, v)) + r2_norm_sq(r2_sub(u, v)) =
        (Real.1 + Real.1) * r2_norm_sq(u) + (Real.1 + Real.1) * r2_norm_sq(v)
} by {
    r2_norm_sq_add_expansion(u, v)
    r2_norm_sq(r2_add(u, v)) = r2_norm_sq(u) + r2_norm_sq(v) + r2_dot(u, v) + r2_dot(u, v)
    r2_norm_sq_sub_expansion(u, v)
    r2_norm_sq(r2_sub(u, v)) = r2_norm_sq(u) + r2_norm_sq(v) - r2_dot(u, v) - r2_dot(u, v)
    r2_norm_sq(r2_add(u, v)) + r2_norm_sq(r2_sub(u, v)) =
        (r2_norm_sq(u) + r2_norm_sq(v) + r2_dot(u, v) + r2_dot(u, v)) +
        (r2_norm_sq(u) + r2_norm_sq(v) - r2_dot(u, v) - r2_dot(u, v))
    r2_norm_sq(u) + r2_norm_sq(v) - r2_dot(u, v) - r2_dot(u, v) =
        r2_norm_sq(u) + r2_norm_sq(v) + -r2_dot(u, v) + -r2_dot(u, v)
    r2_norm_sq(r2_add(u, v)) + r2_norm_sq(r2_sub(u, v)) =
        (r2_norm_sq(u) + r2_norm_sq(v) + r2_dot(u, v) + r2_dot(u, v)) +
        (r2_norm_sq(u) + r2_norm_sq(v) + -r2_dot(u, v) + -r2_dot(u, v))
    r2_norm_sq(u) + r2_norm_sq(v) + r2_dot(u, v) + r2_dot(u, v) =
        (r2_norm_sq(u) + r2_norm_sq(v)) + (r2_dot(u, v) + r2_dot(u, v))
    r2_norm_sq(u) + r2_norm_sq(v) + -r2_dot(u, v) + -r2_dot(u, v) =
        (r2_norm_sq(u) + r2_norm_sq(v)) + (-r2_dot(u, v) + -r2_dot(u, v))
    r2_norm_sq(r2_add(u, v)) + r2_norm_sq(r2_sub(u, v)) =
        ((r2_norm_sq(u) + r2_norm_sq(v)) + (r2_dot(u, v) + r2_dot(u, v))) +
        ((r2_norm_sq(u) + r2_norm_sq(v)) + (-r2_dot(u, v) + -r2_dot(u, v)))
    real_add_pair_rearrange(r2_norm_sq(u) + r2_norm_sq(v), r2_dot(u, v) + r2_dot(u, v), r2_norm_sq(u) + r2_norm_sq(v), -r2_dot(u, v) + -r2_dot(u, v))
    ((r2_norm_sq(u) + r2_norm_sq(v)) + (r2_dot(u, v) + r2_dot(u, v))) +
        ((r2_norm_sq(u) + r2_norm_sq(v)) + (-r2_dot(u, v) + -r2_dot(u, v))) =
        ((r2_norm_sq(u) + r2_norm_sq(v)) + (r2_norm_sq(u) + r2_norm_sq(v))) +
        ((r2_dot(u, v) + r2_dot(u, v)) + (-r2_dot(u, v) + -r2_dot(u, v)))
    real_double_cancel(r2_dot(u, v))
    r2_dot(u, v) + r2_dot(u, v) + (-r2_dot(u, v) + -r2_dot(u, v)) = Real.0
    (r2_dot(u, v) + r2_dot(u, v)) + (-r2_dot(u, v) + -r2_dot(u, v)) = Real.0
    (r2_norm_sq(u) + r2_norm_sq(v)) + (r2_norm_sq(u) + r2_norm_sq(v)) =
        (Real.1 + Real.1) * (r2_norm_sq(u) + r2_norm_sq(v))
    (Real.1 + Real.1) * (r2_norm_sq(u) + r2_norm_sq(v)) =
        (Real.1 + Real.1) * r2_norm_sq(u) + (Real.1 + Real.1) * r2_norm_sq(v)
    ((r2_norm_sq(u) + r2_norm_sq(v)) + (r2_norm_sq(u) + r2_norm_sq(v))) +
        ((r2_dot(u, v) + r2_dot(u, v)) + (-r2_dot(u, v) + -r2_dot(u, v))) =
        (Real.1 + Real.1) * r2_norm_sq(u) + (Real.1 + Real.1) * r2_norm_sq(v) + Real.0
    (Real.1 + Real.1) * r2_norm_sq(u) + (Real.1 + Real.1) * r2_norm_sq(v) + Real.0 =
        (Real.1 + Real.1) * r2_norm_sq(u) + (Real.1 + Real.1) * r2_norm_sq(v)
}

/// Adding points is associative.
theorem r2_add_assoc(a: Pair[Real, Real], b: Pair[Real, Real], c: Pair[Real, Real]) {
    r2_add(r2_add(a, b), c) = r2_add(a, r2_add(b, c))
} by {
    let lhs = r2_add(r2_add(a, b), c)
    let rhs = r2_add(a, r2_add(b, c))
    lhs.first = r2_add(a, b).first + c.first
    r2_add(a, b).first = a.first + b.first
    lhs.first = (a.first + b.first) + c.first
    lhs.second = r2_add(a, b).second + c.second
    r2_add(a, b).second = a.second + b.second
    lhs.second = (a.second + b.second) + c.second
    rhs.first = a.first + r2_add(b, c).first
    r2_add(b, c).first = b.first + c.first
    rhs.first = a.first + (b.first + c.first)
    rhs.second = a.second + r2_add(b, c).second
    r2_add(b, c).second = b.second + c.second
    rhs.second = a.second + (b.second + c.second)
    lhs.first = rhs.first
    lhs.second = rhs.second
    pair_ext(lhs, rhs)
}

/// Adding points is commutative.
theorem r2_add_comm(a: Pair[Real, Real], b: Pair[Real, Real]) {
    r2_add(a, b) = r2_add(b, a)
} by {
    let lhs = r2_add(a, b)
    let rhs = r2_add(b, a)
    lhs.first = a.first + b.first
    lhs.second = a.second + b.second
    rhs.first = b.first + a.first
    rhs.second = b.second + a.second
    a.first + b.first = b.first + a.first
    a.second + b.second = b.second + a.second
    lhs.first = rhs.first
    lhs.second = rhs.second
    pair_ext(lhs, rhs)
}

/// The squared norm of a three-term sum expands completely.
theorem r2_norm_sq_add_add_expand(w: Pair[Real, Real], u: Pair[Real, Real], v: Pair[Real, Real]) {
    r2_norm_sq(r2_add(r2_add(w, u), v)) =
        r2_norm_sq(w) + r2_norm_sq(u) + r2_norm_sq(v) +
        r2_dot(u, v) + r2_dot(u, v) +
        r2_dot(w, u) + r2_dot(w, v) +
        r2_dot(w, u) + r2_dot(w, v)
} by {
    r2_add_assoc(w, u, v)
    r2_add(r2_add(w, u), v) = r2_add(w, r2_add(u, v))
    r2_norm_sq(r2_add(r2_add(w, u), v)) = r2_norm_sq(r2_add(w, r2_add(u, v)))
    r2_norm_sq_add_expansion(w, r2_add(u, v))
    r2_norm_sq(r2_add(w, r2_add(u, v))) =
        r2_norm_sq(w) + r2_norm_sq(r2_add(u, v)) + r2_dot(w, r2_add(u, v)) + r2_dot(w, r2_add(u, v))
    r2_norm_sq_add_expansion(u, v)
    r2_norm_sq(r2_add(u, v)) = r2_norm_sq(u) + r2_norm_sq(v) + r2_dot(u, v) + r2_dot(u, v)
    r2_dot_add_right(w, u, v)
    r2_dot(w, r2_add(u, v)) = r2_dot(w, u) + r2_dot(w, v)
    r2_norm_sq(r2_add(w, r2_add(u, v))) =
        r2_norm_sq(w) + r2_norm_sq(r2_add(u, v)) + r2_dot(w, r2_add(u, v)) + r2_dot(w, r2_add(u, v))
    r2_norm_sq(w) + r2_norm_sq(r2_add(u, v)) + r2_dot(w, r2_add(u, v)) + r2_dot(w, r2_add(u, v)) =
        r2_norm_sq(w) + (r2_norm_sq(u) + r2_norm_sq(v) + r2_dot(u, v) + r2_dot(u, v)) +
        r2_dot(w, r2_add(u, v)) + r2_dot(w, r2_add(u, v))
    r2_norm_sq(w) + (r2_norm_sq(u) + r2_norm_sq(v) + r2_dot(u, v) + r2_dot(u, v)) +
        r2_dot(w, r2_add(u, v)) + r2_dot(w, r2_add(u, v)) =
        r2_norm_sq(w) + (r2_norm_sq(u) + r2_norm_sq(v) + r2_dot(u, v) + r2_dot(u, v)) +
        (r2_dot(w, u) + r2_dot(w, v)) + r2_dot(w, r2_add(u, v))
    r2_norm_sq(w) + (r2_norm_sq(u) + r2_norm_sq(v) + r2_dot(u, v) + r2_dot(u, v)) +
        (r2_dot(w, u) + r2_dot(w, v)) + r2_dot(w, r2_add(u, v)) =
        r2_norm_sq(w) + (r2_norm_sq(u) + r2_norm_sq(v) + r2_dot(u, v) + r2_dot(u, v)) +
        (r2_dot(w, u) + r2_dot(w, v)) + (r2_dot(w, u) + r2_dot(w, v))
    r2_norm_sq(w) + (r2_norm_sq(u) + r2_norm_sq(v) + r2_dot(u, v) + r2_dot(u, v)) +
        (r2_dot(w, u) + r2_dot(w, v)) + (r2_dot(w, u) + r2_dot(w, v)) =
        (r2_norm_sq(w) + (r2_norm_sq(u) + r2_norm_sq(v) + r2_dot(u, v) + r2_dot(u, v))) +
        (r2_dot(w, u) + r2_dot(w, v)) + (r2_dot(w, u) + r2_dot(w, v))
    r2_norm_sq(w) + (r2_norm_sq(u) + r2_norm_sq(v) + r2_dot(u, v) + r2_dot(u, v)) =
        (r2_norm_sq(w) + (r2_norm_sq(u) + r2_norm_sq(v)) + r2_dot(u, v)) + r2_dot(u, v)
    (r2_norm_sq(w) + (r2_norm_sq(u) + r2_norm_sq(v)) + r2_dot(u, v)) + r2_dot(u, v) =
        r2_norm_sq(w) + r2_norm_sq(u) + r2_norm_sq(v) + r2_dot(u, v) + r2_dot(u, v)
    (r2_norm_sq(w) + r2_norm_sq(u) + r2_norm_sq(v) + r2_dot(u, v) + r2_dot(u, v)) +
        (r2_dot(w, u) + r2_dot(w, v)) + (r2_dot(w, u) + r2_dot(w, v)) =
        r2_norm_sq(w) + r2_norm_sq(u) + r2_norm_sq(v) + r2_dot(u, v) + r2_dot(u, v) +
        r2_dot(w, u) + r2_dot(w, v) + r2_dot(w, u) + r2_dot(w, v)
    r2_norm_sq(r2_add(r2_add(w, u), v)) =
        r2_norm_sq(w) + r2_norm_sq(u) + r2_norm_sq(v) + r2_dot(u, v) + r2_dot(u, v) +
        r2_dot(w, u) + r2_dot(w, v) + r2_dot(w, u) + r2_dot(w, v)
}

/// The first coordinate of the midpoint negation identity.
theorem r2_midpoint_sub_neg_first(b: Pair[Real, Real], c: Pair[Real, Real], m: Pair[Real, Real]) {
    r2_add(b, c) = r2_add(m, m) implies
    r2_sub(c, m).first = r2_neg(r2_sub(b, m)).first
} by {
    if r2_add(b, c) = r2_add(m, m) {
        r2_add(b, c).first = r2_add(m, m).first
        r2_add(b, c).first = b.first + c.first
        r2_add(m, m).first = m.first + m.first
        b.first + c.first = m.first + m.first
        real_double_eq_add_self(m.first)
        (Real.1 + Real.1) * m.first = m.first + m.first
        b.first + c.first = (Real.1 + Real.1) * m.first
        real_neg_sub_swap(b.first, c.first, m.first)
        c.first - m.first = -(b.first - m.first)
        r2_sub(c, m).first = c.first - m.first
        r2_neg(r2_sub(b, m)).first = -r2_sub(b, m).first
        r2_sub(b, m).first = b.first - m.first
        r2_neg(r2_sub(b, m)).first = -(b.first - m.first)
        r2_sub(c, m).first = r2_neg(r2_sub(b, m)).first
    }
}

/// The second coordinate of the midpoint negation identity.
theorem r2_midpoint_sub_neg_second(b: Pair[Real, Real], c: Pair[Real, Real], m: Pair[Real, Real]) {
    r2_add(b, c) = r2_add(m, m) implies
    r2_sub(c, m).second = r2_neg(r2_sub(b, m)).second
} by {
    if r2_add(b, c) = r2_add(m, m) {
        r2_add(b, c).second = r2_add(m, m).second
        r2_add(b, c).second = b.second + c.second
        r2_add(m, m).second = m.second + m.second
        b.second + c.second = m.second + m.second
        real_double_eq_add_self(m.second)
        (Real.1 + Real.1) * m.second = m.second + m.second
        b.second + c.second = (Real.1 + Real.1) * m.second
        real_neg_sub_swap(b.second, c.second, m.second)
        c.second - m.second = -(b.second - m.second)
        r2_sub(c, m).second = c.second - m.second
        r2_neg(r2_sub(b, m)).second = -r2_sub(b, m).second
        r2_sub(b, m).second = b.second - m.second
        r2_neg(r2_sub(b, m)).second = -(b.second - m.second)
        r2_sub(c, m).second = r2_neg(r2_sub(b, m)).second
    }
}

/// From `b + c = 2m`, the displacement from the midpoint to `c` is the
/// negation of the displacement from the midpoint to `b`.
theorem r2_midpoint_sub_neg(b: Pair[Real, Real], c: Pair[Real, Real], m: Pair[Real, Real]) {
    r2_add(b, c) = r2_add(m, m) implies
    r2_sub(c, m) = r2_neg(r2_sub(b, m))
} by {
    if r2_add(b, c) = r2_add(m, m) {
        r2_midpoint_sub_neg_first(b, c, m)
        r2_sub(c, m).first = r2_neg(r2_sub(b, m)).first
        r2_midpoint_sub_neg_second(b, c, m)
        r2_sub(c, m).second = r2_neg(r2_sub(b, m)).second
        r2_sub(c, m) = r2_neg(r2_sub(b, m))
    }
}

/// The first coordinate of the midpoint swap identity.
theorem r2_midpoint_sub_swap_first(a: Pair[Real, Real], b: Pair[Real, Real], m: Pair[Real, Real]) {
    r2_add(a, b) = r2_add(m, m) implies
    r2_sub(m, b).first = r2_sub(a, m).first
} by {
    if r2_add(a, b) = r2_add(m, m) {
        r2_add(a, b).first = r2_add(m, m).first
        r2_add(a, b).first = a.first + b.first
        r2_add(m, m).first = m.first + m.first
        a.first + b.first = m.first + m.first
        real_double_eq_add_self(m.first)
        (Real.1 + Real.1) * m.first = m.first + m.first
        a.first + b.first = (Real.1 + Real.1) * m.first
        real_mid_sub_swap(a.first, b.first, m.first)
        m.first - b.first = a.first - m.first
        r2_sub(m, b).first = m.first - b.first
        r2_sub(a, m).first = a.first - m.first
        r2_sub(m, b).first = r2_sub(a, m).first
    }
}

/// The second coordinate of the midpoint swap identity.
theorem r2_midpoint_sub_swap_second(a: Pair[Real, Real], b: Pair[Real, Real], m: Pair[Real, Real]) {
    r2_add(a, b) = r2_add(m, m) implies
    r2_sub(m, b).second = r2_sub(a, m).second
} by {
    if r2_add(a, b) = r2_add(m, m) {
        r2_add(a, b).second = r2_add(m, m).second
        r2_add(a, b).second = a.second + b.second
        r2_add(m, m).second = m.second + m.second
        a.second + b.second = m.second + m.second
        real_double_eq_add_self(m.second)
        (Real.1 + Real.1) * m.second = m.second + m.second
        a.second + b.second = (Real.1 + Real.1) * m.second
        real_mid_sub_swap(a.second, b.second, m.second)
        m.second - b.second = a.second - m.second
        r2_sub(m, b).second = m.second - b.second
        r2_sub(a, m).second = a.second - m.second
        r2_sub(m, b).second = r2_sub(a, m).second
    }
}

/// From `a + b = 2m`, the displacement from `b` to the midpoint is the
/// displacement from the midpoint to `a`.
theorem r2_midpoint_sub_swap(a: Pair[Real, Real], b: Pair[Real, Real], m: Pair[Real, Real]) {
    r2_add(a, b) = r2_add(m, m) implies
    r2_sub(m, b) = r2_sub(a, m)
} by {
    if r2_add(a, b) = r2_add(m, m) {
        r2_midpoint_sub_swap_first(a, b, m)
        r2_sub(m, b).first = r2_sub(a, m).first
        r2_midpoint_sub_swap_second(a, b, m)
        r2_sub(m, b).second = r2_sub(a, m).second
        r2_sub(m, b) = r2_sub(a, m)
    }
}

// ---------------------------------------------------------------------------
// Scalar multiplication, the cross product, and doubling, for the second
// batch of theorems1000 theorems.
// ---------------------------------------------------------------------------

/// Scalar multiplication of a coordinate pair.
define r2_smul(k: Real, a: Pair[Real, Real]) -> Pair[Real, Real] {
    Pair.new(k * a.first, k * a.second)
}

/// The signed cross product (determinant) of two coordinate pairs.
define r2_cross(a: Pair[Real, Real], b: Pair[Real, Real]) -> Real {
    a.first * b.second - a.second * b.first
}

/// A coordinate pair is determined by its two coordinates.
theorem r2_pair_ext(a: Pair[Real, Real], b: Pair[Real, Real]) {
    a.first = b.first and a.second = b.second implies a = b
} by {
    if a.first = b.first and a.second = b.second {
        pair_ext(a, b)
    }
}

/// The cross product of a pair with itself is zero.
theorem r2_cross_self_zero(a: Pair[Real, Real]) {
    r2_cross(a, a) = Real.0
} by {
    r2_cross(a, a) = a.first * a.second - a.second * a.first
    a.second * a.first = a.first * a.second
    a.first * a.second - a.first * a.second = Real.0
}

/// Swapping the arguments negates a cross product.
theorem r2_cross_swap(a: Pair[Real, Real], b: Pair[Real, Real]) {
    r2_cross(a, b) = -r2_cross(b, a)
} by {
    real_sub_reverse_neg(a.first * b.second, a.second * b.first)
    a.first * b.second - a.second * b.first = -(a.second * b.first - a.first * b.second)
    r2_cross(b, a) = b.first * a.second - b.second * a.first
    b.first * a.second = a.second * b.first
    b.second * a.first = a.first * b.second
    r2_cross(b, a) = a.second * b.first - a.first * b.second
    r2_cross(a, b) = -r2_cross(b, a)
}

/// Negating the left argument negates a cross product.
theorem r2_cross_neg_left(a: Pair[Real, Real], b: Pair[Real, Real]) {
    r2_cross(r2_neg(a), b) = -r2_cross(a, b)
} by {
    let x = a.first * b.second
    let y = a.second * b.first
    r2_cross(r2_neg(a), b) = r2_neg(a).first * b.second - r2_neg(a).second * b.first
    r2_neg(a).first = -a.first
    r2_neg(a).second = -a.second
    r2_cross(r2_neg(a), b) = (-a.first) * b.second - (-a.second) * b.first
    (-a.first) * b.second = -x
    (-a.second) * b.first = -y
    -x - -y = y - x
    real_sub_reverse_neg(y, x)
    y - x = -(x - y)
    r2_cross(a, b) = x - y
    r2_cross(r2_neg(a), b) = -r2_cross(a, b)
}

/// Negating the right argument negates a cross product.
theorem r2_cross_neg_right(a: Pair[Real, Real], b: Pair[Real, Real]) {
    r2_cross(a, r2_neg(b)) = -r2_cross(a, b)
} by {
    let x = a.first * b.second
    let y = a.second * b.first
    r2_cross(a, r2_neg(b)) = a.first * r2_neg(b).second - a.second * r2_neg(b).first
    r2_neg(b).second = -b.second
    r2_neg(b).first = -b.first
    r2_cross(a, r2_neg(b)) = a.first * (-b.second) - a.second * (-b.first)
    a.first * (-b.second) = -x
    a.second * (-b.first) = -y
    -x - -y = y - x
    real_sub_reverse_neg(y, x)
    y - x = -(x - y)
    r2_cross(a, b) = x - y
    r2_cross(a, r2_neg(b)) = -r2_cross(a, b)
}

/// The cross product distributes over addition on the left.
theorem r2_cross_add_left(a: Pair[Real, Real], b: Pair[Real, Real], c: Pair[Real, Real]) {
    r2_cross(r2_add(a, b), c) = r2_cross(a, c) + r2_cross(b, c)
} by {
    let x = a.first * c.second
    let y = a.second * c.first
    let z = b.first * c.second
    let w = b.second * c.first
    r2_cross(r2_add(a, b), c) = r2_add(a, b).first * c.second - r2_add(a, b).second * c.first
    r2_add(a, b).first = a.first + b.first
    r2_add(a, b).second = a.second + b.second
    r2_cross(r2_add(a, b), c) = (a.first + b.first) * c.second - (a.second + b.second) * c.first
    (a.first + b.first) * c.second = x + z
    (a.second + b.second) * c.first = y + w
    r2_cross(r2_add(a, b), c) = (x + z) - (y + w)
    real_sub_add_distrib(x, y, z, w)
    (x - y) + (z - w) = (x + z) - (y + w)
    r2_cross(a, c) = x - y
    r2_cross(b, c) = z - w
    r2_cross(r2_add(a, b), c) = r2_cross(a, c) + r2_cross(b, c)
}

/// The cross product distributes over addition on the right.
theorem r2_cross_add_right(a: Pair[Real, Real], b: Pair[Real, Real], c: Pair[Real, Real]) {
    r2_cross(a, r2_add(b, c)) = r2_cross(a, b) + r2_cross(a, c)
} by {
    let x = a.first * b.second
    let y = a.second * b.first
    let z = a.first * c.second
    let w = a.second * c.first
    r2_cross(a, r2_add(b, c)) = a.first * r2_add(b, c).second - a.second * r2_add(b, c).first
    r2_add(b, c).second = b.second + c.second
    r2_add(b, c).first = b.first + c.first
    r2_cross(a, r2_add(b, c)) = a.first * (b.second + c.second) - a.second * (b.first + c.first)
    a.first * (b.second + c.second) = x + z
    a.second * (b.first + c.first) = y + w
    r2_cross(a, r2_add(b, c)) = (x + z) - (y + w)
    real_sub_add_distrib(x, y, z, w)
    (x - y) + (z - w) = (x + z) - (y + w)
    r2_cross(a, b) = x - y
    r2_cross(a, c) = z - w
    r2_cross(a, r2_add(b, c)) = r2_cross(a, b) + r2_cross(a, c)
}

/// A scalar may be factored out of the left argument of a cross product.
theorem r2_cross_smul_left(k: Real, a: Pair[Real, Real], b: Pair[Real, Real]) {
    r2_cross(r2_smul(k, a), b) = k * r2_cross(a, b)
} by {
    let x = a.first * b.second
    let y = a.second * b.first
    let sa = r2_smul(k, a)
    sa.first = k * a.first
    sa.second = k * a.second
    r2_cross(sa, b) = sa.first * b.second - sa.second * b.first
    r2_cross(sa, b) = (k * a.first) * b.second - (k * a.second) * b.first
    (k * a.first) * b.second = k * x
    (k * a.second) * b.first = k * y
    k * x - k * y = k * (x - y)
    r2_cross(a, b) = x - y
    r2_cross(sa, b) = k * r2_cross(a, b)
    r2_cross(r2_smul(k, a), b) = r2_cross(sa, b)
    r2_cross(r2_smul(k, a), b) = k * r2_cross(a, b)
}

/// A scalar may be factored out of the right argument of a cross product.
theorem r2_cross_smul_right(k: Real, a: Pair[Real, Real], b: Pair[Real, Real]) {
    r2_cross(a, r2_smul(k, b)) = k * r2_cross(a, b)
} by {
    let x = a.first * b.second
    let y = a.second * b.first
    let sb = r2_smul(k, b)
    sb.second = k * b.second
    sb.first = k * b.first
    r2_cross(a, sb) = a.first * sb.second - a.second * sb.first
    r2_cross(a, sb) = a.first * (k * b.second) - a.second * (k * b.first)
    a.first * (k * b.second) = k * x
    a.second * (k * b.first) = k * y
    k * x - k * y = k * (x - y)
    r2_cross(a, b) = x - y
    r2_cross(a, sb) = k * r2_cross(a, b)
    r2_cross(a, r2_smul(k, b)) = r2_cross(a, sb)
    r2_cross(a, r2_smul(k, b)) = k * r2_cross(a, b)
}

/// The cross product distributes over subtraction on the left.
theorem r2_cross_sub_left(a: Pair[Real, Real], b: Pair[Real, Real], c: Pair[Real, Real]) {
    r2_cross(r2_sub(a, b), c) = r2_cross(a, c) - r2_cross(b, c)
} by {
    r2_sub_eq_add_neg(a, b)
    r2_sub(a, b) = r2_add(a, r2_neg(b))
    r2_cross(r2_sub(a, b), c) = r2_cross(r2_add(a, r2_neg(b)), c)
    r2_cross_add_left(a, r2_neg(b), c)
    r2_cross(r2_add(a, r2_neg(b)), c) = r2_cross(a, c) + r2_cross(r2_neg(b), c)
    r2_cross_neg_left(b, c)
    r2_cross(r2_neg(b), c) = -r2_cross(b, c)
    r2_cross(a, c) + -r2_cross(b, c) = r2_cross(a, c) - r2_cross(b, c)
    r2_cross(r2_sub(a, b), c) = r2_cross(a, c) - r2_cross(b, c)
}

/// The cross product distributes over subtraction on the right.
theorem r2_cross_sub_right(a: Pair[Real, Real], b: Pair[Real, Real], c: Pair[Real, Real]) {
    r2_cross(a, r2_sub(b, c)) = r2_cross(a, b) - r2_cross(a, c)
} by {
    r2_sub_eq_add_neg(b, c)
    r2_sub(b, c) = r2_add(b, r2_neg(c))
    r2_cross(a, r2_sub(b, c)) = r2_cross(a, r2_add(b, r2_neg(c)))
    r2_cross_add_right(a, b, r2_neg(c))
    r2_cross(a, r2_add(b, r2_neg(c))) = r2_cross(a, b) + r2_cross(a, r2_neg(c))
    r2_cross_neg_right(a, c)
    r2_cross(a, r2_neg(c)) = -r2_cross(a, c)
    r2_cross(a, b) + -r2_cross(a, c) = r2_cross(a, b) - r2_cross(a, c)
    r2_cross(a, r2_sub(b, c)) = r2_cross(a, b) - r2_cross(a, c)
}

/// Scalar multiplication distributes over point addition.
theorem r2_smul_add_distrib(k: Real, a: Pair[Real, Real], b: Pair[Real, Real]) {
    r2_smul(k, r2_add(a, b)) = r2_add(r2_smul(k, a), r2_smul(k, b))
} by {
    let lhs = r2_smul(k, r2_add(a, b))
    let sa = r2_smul(k, a)
    let sb = r2_smul(k, b)
    let rhs = r2_add(sa, sb)
    lhs.first = k * r2_add(a, b).first
    r2_add(a, b).first = a.first + b.first
    lhs.first = k * (a.first + b.first)
    k * (a.first + b.first) = k * a.first + k * b.first
    lhs.second = k * r2_add(a, b).second
    r2_add(a, b).second = a.second + b.second
    lhs.second = k * (a.second + b.second)
    k * (a.second + b.second) = k * a.second + k * b.second
    sa.first = k * a.first
    sa.second = k * a.second
    sb.first = k * b.first
    sb.second = k * b.second
    rhs.first = sa.first + sb.first
    rhs.first = k * a.first + k * b.first
    rhs.second = sa.second + sb.second
    rhs.second = k * a.second + k * b.second
    lhs.first = rhs.first
    lhs.second = rhs.second
    pair_ext(lhs, rhs)
}

/// Scalar multiplication distributes over point subtraction.
theorem r2_smul_sub_distrib(k: Real, a: Pair[Real, Real], b: Pair[Real, Real]) {
    r2_smul(k, r2_sub(a, b)) = r2_sub(r2_smul(k, a), r2_smul(k, b))
} by {
    let lhs = r2_smul(k, r2_sub(a, b))
    let sa = r2_smul(k, a)
    let sb = r2_smul(k, b)
    let rhs = r2_sub(sa, sb)
    lhs.first = k * r2_sub(a, b).first
    r2_sub(a, b).first = a.first - b.first
    lhs.first = k * (a.first - b.first)
    k * (a.first - b.first) = k * a.first - k * b.first
    lhs.second = k * r2_sub(a, b).second
    r2_sub(a, b).second = a.second - b.second
    lhs.second = k * (a.second - b.second)
    k * (a.second - b.second) = k * a.second - k * b.second
    sa.first = k * a.first
    sa.second = k * a.second
    sb.first = k * b.first
    sb.second = k * b.second
    rhs.first = sa.first - sb.first
    rhs.first = k * a.first - k * b.first
    rhs.second = sa.second - sb.second
    rhs.second = k * a.second - k * b.second
    lhs.first = rhs.first
    lhs.second = rhs.second
    pair_ext(lhs, rhs)
}

/// A scalar times the negation of a point is the negation of the product.
theorem r2_smul_neg(k: Real, a: Pair[Real, Real]) {
    r2_smul(k, r2_neg(a)) = r2_neg(r2_smul(k, a))
} by {
    let lhs = r2_smul(k, r2_neg(a))
    let sa = r2_smul(k, a)
    let rhs = r2_neg(sa)
    lhs.first = k * r2_neg(a).first
    r2_neg(a).first = -a.first
    lhs.first = k * (-a.first)
    k * (-a.first) = -(k * a.first)
    sa.first = k * a.first
    rhs.first = -sa.first
    rhs.first = -(k * a.first)
    lhs.first = rhs.first
    lhs.second = k * r2_neg(a).second
    r2_neg(a).second = -a.second
    lhs.second = k * (-a.second)
    k * (-a.second) = -(k * a.second)
    sa.second = k * a.second
    rhs.second = -sa.second
    rhs.second = -(k * a.second)
    lhs.second = rhs.second
    pair_ext(lhs, rhs)
}

/// Scaling a pair by one leaves it unchanged.
theorem r2_smul_one(a: Pair[Real, Real]) {
    r2_smul(Real.1, a) = a
} by {
    let lhs = r2_smul(Real.1, a)
    lhs.first = Real.1 * a.first
    Real.1 * a.first = a.first
    lhs.second = Real.1 * a.second
    Real.1 * a.second = a.second
    pair_ext(lhs, a)
}

/// The dot product is symmetric.
theorem r2_dot_comm(a: Pair[Real, Real], b: Pair[Real, Real]) {
    r2_dot(a, b) = r2_dot(b, a)
}

/// A scalar may be factored out of the left argument of a dot product.
theorem r2_dot_smul_left(k: Real, a: Pair[Real, Real], b: Pair[Real, Real]) {
    r2_dot(r2_smul(k, a), b) = k * r2_dot(a, b)
} by {
    let sa = r2_smul(k, a)
    sa.first = k * a.first
    sa.second = k * a.second
    r2_dot(sa, b) = sa.first * b.first + sa.second * b.second
    r2_dot(sa, b) = (k * a.first) * b.first + (k * a.second) * b.second
    (k * a.first) * b.first = k * (a.first * b.first)
    (k * a.second) * b.second = k * (a.second * b.second)
    k * (a.first * b.first) + k * (a.second * b.second) = k * (a.first * b.first + a.second * b.second)
    r2_dot(a, b) = a.first * b.first + a.second * b.second
    r2_dot(sa, b) = k * r2_dot(a, b)
    r2_dot(r2_smul(k, a), b) = r2_dot(sa, b)
    r2_dot(r2_smul(k, a), b) = k * r2_dot(a, b)
}

/// A scalar may be factored out of the right argument of a dot product.
theorem r2_dot_smul_right(k: Real, a: Pair[Real, Real], b: Pair[Real, Real]) {
    r2_dot(a, r2_smul(k, b)) = k * r2_dot(a, b)
} by {
    let sb = r2_smul(k, b)
    sb.first = k * b.first
    sb.second = k * b.second
    r2_dot(a, sb) = a.first * sb.first + a.second * sb.second
    r2_dot(a, sb) = a.first * (k * b.first) + a.second * (k * b.second)
    a.first * (k * b.first) = k * (a.first * b.first)
    a.second * (k * b.second) = k * (a.second * b.second)
    k * (a.first * b.first) + k * (a.second * b.second) = k * (a.first * b.first + a.second * b.second)
    r2_dot(a, b) = a.first * b.first + a.second * b.second
    r2_dot(a, sb) = k * r2_dot(a, b)
    r2_dot(a, r2_smul(k, b)) = r2_dot(a, sb)
    r2_dot(a, r2_smul(k, b)) = k * r2_dot(a, b)
}

/// The squared norm of a scaled pair is the scaled squared norm.
theorem r2_norm_sq_smul(k: Real, a: Pair[Real, Real]) {
    r2_norm_sq(r2_smul(k, a)) = k * k * r2_norm_sq(a)
} by {
    r2_norm_sq(r2_smul(k, a)) = r2_dot(r2_smul(k, a), r2_smul(k, a))
    r2_dot_smul_left(k, a, r2_smul(k, a))
    r2_dot(r2_smul(k, a), r2_smul(k, a)) = k * r2_dot(a, r2_smul(k, a))
    r2_dot_smul_right(k, a, a)
    r2_dot(a, r2_smul(k, a)) = k * r2_dot(a, a)
    r2_norm_sq(a) = r2_dot(a, a)
    r2_norm_sq(r2_smul(k, a)) = k * (k * r2_norm_sq(a))
    k * (k * r2_norm_sq(a)) = k * k * r2_norm_sq(a)
    r2_norm_sq(r2_smul(k, a)) = k * k * r2_norm_sq(a)
}

/// The squared norm of a negated pair is the squared norm.
theorem r2_norm_sq_neg(a: Pair[Real, Real]) {
    r2_norm_sq(r2_neg(a)) = r2_norm_sq(a)
} by {
    r2_norm_sq(r2_neg(a)) = r2_dot(r2_neg(a), r2_neg(a))
    r2_dot_neg_left(a, r2_neg(a))
    r2_dot(r2_neg(a), r2_neg(a)) = -r2_dot(a, r2_neg(a))
    r2_dot_neg_right(a, a)
    r2_dot(a, r2_neg(a)) = -r2_dot(a, a)
    -(-r2_dot(a, a)) = r2_dot(a, a)
    r2_norm_sq(a) = r2_dot(a, a)
    r2_norm_sq(r2_neg(a)) = r2_norm_sq(a)
}

/// The squared norm of a doubled pair is four times the squared norm.
theorem r2_norm_sq_double(a: Pair[Real, Real]) {
    r2_norm_sq(r2_add(a, a)) = (Real.1 + Real.1) * ((Real.1 + Real.1) * r2_norm_sq(a))
} by {
    r2_norm_sq_add_expansion(a, a)
    r2_norm_sq(r2_add(a, a)) = r2_norm_sq(a) + r2_norm_sq(a) + r2_dot(a, a) + r2_dot(a, a)
    r2_norm_sq(a) = r2_dot(a, a)
    r2_norm_sq(a) + r2_norm_sq(a) + r2_norm_sq(a) + r2_norm_sq(a) =
        (Real.1 + Real.1) * ((Real.1 + Real.1) * r2_norm_sq(a))
    r2_norm_sq(r2_add(a, a)) = (Real.1 + Real.1) * ((Real.1 + Real.1) * r2_norm_sq(a))
}

/// Doubling distributes over point subtraction.
theorem r2_double_sub_distrib(a: Pair[Real, Real], b: Pair[Real, Real]) {
    r2_add(r2_sub(a, b), r2_sub(a, b)) = r2_sub(r2_add(a, a), r2_add(b, b))
} by {
    let lhs = r2_add(r2_sub(a, b), r2_sub(a, b))
    let rhs = r2_sub(r2_add(a, a), r2_add(b, b))
    lhs.first = r2_sub(a, b).first + r2_sub(a, b).first
    r2_sub(a, b).first = a.first - b.first
    lhs.first = (a.first - b.first) + (a.first - b.first)
    rhs.first = r2_add(a, a).first - r2_add(b, b).first
    r2_add(a, a).first = a.first + a.first
    r2_add(b, b).first = b.first + b.first
    rhs.first = (a.first + a.first) - (b.first + b.first)
    (a.first - b.first) + (a.first - b.first) = (a.first + a.first) - (b.first + b.first)
    lhs.first = rhs.first
    lhs.second = r2_sub(a, b).second + r2_sub(a, b).second
    r2_sub(a, b).second = a.second - b.second
    lhs.second = (a.second - b.second) + (a.second - b.second)
    rhs.second = r2_add(a, a).second - r2_add(b, b).second
    r2_add(a, a).second = a.second + a.second
    r2_add(b, b).second = b.second + b.second
    rhs.second = (a.second + a.second) - (b.second + b.second)
    (a.second - b.second) + (a.second - b.second) = (a.second + a.second) - (b.second + b.second)
    lhs.second = rhs.second
    pair_ext(lhs, rhs)
}

/// Tripling distributes over point subtraction.
theorem r2_triple_sub_distrib(a: Pair[Real, Real], b: Pair[Real, Real]) {
    r2_add(r2_sub(a, b), r2_add(r2_sub(a, b), r2_sub(a, b))) =
        r2_sub(r2_add(a, r2_add(a, a)), r2_add(b, r2_add(b, b)))
} by {
    let lhs = r2_add(r2_sub(a, b), r2_add(r2_sub(a, b), r2_sub(a, b)))
    let rhs = r2_sub(r2_add(a, r2_add(a, a)), r2_add(b, r2_add(b, b)))
    lhs.first = r2_sub(a, b).first + r2_add(r2_sub(a, b), r2_sub(a, b)).first
    r2_sub(a, b).first = a.first - b.first
    r2_add(r2_sub(a, b), r2_sub(a, b)).first = r2_sub(a, b).first + r2_sub(a, b).first
    r2_add(r2_sub(a, b), r2_sub(a, b)).first = (a.first - b.first) + (a.first - b.first)
    lhs.first = (a.first - b.first) + ((a.first - b.first) + (a.first - b.first))
    rhs.first = r2_add(a, r2_add(a, a)).first - r2_add(b, r2_add(b, b)).first
    r2_add(a, r2_add(a, a)).first = a.first + r2_add(a, a).first
    r2_add(a, a).first = a.first + a.first
    r2_add(a, r2_add(a, a)).first = a.first + (a.first + a.first)
    r2_add(b, r2_add(b, b)).first = b.first + r2_add(b, b).first
    r2_add(b, b).first = b.first + b.first
    r2_add(b, r2_add(b, b)).first = b.first + (b.first + b.first)
    rhs.first = (a.first + (a.first + a.first)) - (b.first + (b.first + b.first))
    (a.first - b.first) + ((a.first - b.first) + (a.first - b.first)) =
        (a.first + (a.first + a.first)) - (b.first + (b.first + b.first))
    lhs.first = rhs.first
    lhs.second = r2_sub(a, b).second + r2_add(r2_sub(a, b), r2_sub(a, b)).second
    r2_sub(a, b).second = a.second - b.second
    r2_add(r2_sub(a, b), r2_sub(a, b)).second = r2_sub(a, b).second + r2_sub(a, b).second
    r2_add(r2_sub(a, b), r2_sub(a, b)).second = (a.second - b.second) + (a.second - b.second)
    lhs.second = (a.second - b.second) + ((a.second - b.second) + (a.second - b.second))
    rhs.second = r2_add(a, r2_add(a, a)).second - r2_add(b, r2_add(b, b)).second
    r2_add(a, r2_add(a, a)).second = a.second + r2_add(a, a).second
    r2_add(a, a).second = a.second + a.second
    r2_add(a, r2_add(a, a)).second = a.second + (a.second + a.second)
    r2_add(b, r2_add(b, b)).second = b.second + r2_add(b, b).second
    r2_add(b, b).second = b.second + b.second
    r2_add(b, r2_add(b, b)).second = b.second + (b.second + b.second)
    rhs.second = (a.second + (a.second + a.second)) - (b.second + (b.second + b.second))
    (a.second - b.second) + ((a.second - b.second) + (a.second - b.second)) =
        (a.second + (a.second + a.second)) - (b.second + (b.second + b.second))
    lhs.second = rhs.second
    pair_ext(lhs, rhs)
}

/// A sum of two differences is the difference of the paired sums.
theorem r2_sub_add_pair_distrib(a: Pair[Real, Real], b: Pair[Real, Real], c: Pair[Real, Real], d: Pair[Real, Real]) {
    r2_sub(r2_add(a, b), r2_add(c, d)) = r2_add(r2_sub(a, c), r2_sub(b, d))
} by {
    let lhs = r2_sub(r2_add(a, b), r2_add(c, d))
    let rhs = r2_add(r2_sub(a, c), r2_sub(b, d))
    lhs.first = r2_add(a, b).first - r2_add(c, d).first
    r2_add(a, b).first = a.first + b.first
    r2_add(c, d).first = c.first + d.first
    lhs.first = (a.first + b.first) - (c.first + d.first)
    rhs.first = r2_sub(a, c).first + r2_sub(b, d).first
    r2_sub(a, c).first = a.first - c.first
    r2_sub(b, d).first = b.first - d.first
    rhs.first = (a.first - c.first) + (b.first - d.first)
    real_sub_add_distrib(a.first, c.first, b.first, d.first)
    (a.first - c.first) + (b.first - d.first) = (a.first + b.first) - (c.first + d.first)
    lhs.first = rhs.first
    lhs.second = r2_add(a, b).second - r2_add(c, d).second
    r2_add(a, b).second = a.second + b.second
    r2_add(c, d).second = c.second + d.second
    lhs.second = (a.second + b.second) - (c.second + d.second)
    rhs.second = r2_sub(a, c).second + r2_sub(b, d).second
    r2_sub(a, c).second = a.second - c.second
    r2_sub(b, d).second = b.second - d.second
    rhs.second = (a.second - c.second) + (b.second - d.second)
    real_sub_add_distrib(a.second, c.second, b.second, d.second)
    (a.second - c.second) + (b.second - d.second) = (a.second + b.second) - (c.second + d.second)
    lhs.second = rhs.second
    pair_ext(lhs, rhs)
}

/// Pairing a sum of four doubled terms may be regrouped.
theorem r2_double_pair_rearrange(a: Pair[Real, Real], b: Pair[Real, Real]) {
    r2_add(r2_add(a, a), r2_add(b, b)) = r2_add(r2_add(a, b), r2_add(a, b))
} by {
    let lhs = r2_add(r2_add(a, a), r2_add(b, b))
    let rhs = r2_add(r2_add(a, b), r2_add(a, b))
    lhs.first = r2_add(a, a).first + r2_add(b, b).first
    r2_add(a, a).first = a.first + a.first
    r2_add(b, b).first = b.first + b.first
    lhs.first = (a.first + a.first) + (b.first + b.first)
    rhs.first = r2_add(a, b).first + r2_add(a, b).first
    r2_add(a, b).first = a.first + b.first
    rhs.first = (a.first + b.first) + (a.first + b.first)
    real_double_pair_rearrange(a.first, b.first)
    (a.first + a.first) + (b.first + b.first) = (a.first + b.first) + (a.first + b.first)
    lhs.first = rhs.first
    lhs.second = r2_add(a, a).second + r2_add(b, b).second
    r2_add(a, a).second = a.second + a.second
    r2_add(b, b).second = b.second + b.second
    lhs.second = (a.second + a.second) + (b.second + b.second)
    rhs.second = r2_add(a, b).second + r2_add(a, b).second
    r2_add(a, b).second = a.second + b.second
    rhs.second = (a.second + b.second) + (a.second + b.second)
    real_double_pair_rearrange(a.second, b.second)
    (a.second + a.second) + (b.second + b.second) = (a.second + b.second) + (a.second + b.second)
    lhs.second = rhs.second
    pair_ext(lhs, rhs)
}

/// Two is a nonzero real.
theorem real_two_neq_zero {
    Real.1 + Real.1 != Real.0
} by {
    Real.0 < Real.1
    Real.0 + Real.0 < Real.1 + Real.1
}

/// A real doubled equality cancels.
theorem real_double_eq_double(x: Real, y: Real) {
    x + x = y + y implies x = y
} by {
    if x + x = y + y {
        (x + x) - (y + y) = Real.0
        real_sub_add_distrib(x, y, x, y)
        (x - y) + (x - y) = (x + x) - (y + y)
        (x - y) + (x - y) = Real.0
        real_double_eq_add_self(x - y)
        (Real.1 + Real.1) * (x - y) = (x - y) + (x - y)
        (Real.1 + Real.1) * (x - y) = Real.0
        Real.0 = (Real.1 + Real.1) * (x - y)
        real_two_neq_zero
        Real.1 + Real.1 != Real.0
        mul_left_cancel(Real.0, Real.1 + Real.1, x - y)
        Real.0 / (Real.1 + Real.1) = x - y
        Real.0 / (Real.1 + Real.1) = Real.0
        x - y = Real.0
        x = y
    }
}

/// A doubled point equality cancels.
theorem r2_double_eq_double(a: Pair[Real, Real], b: Pair[Real, Real]) {
    r2_add(a, a) = r2_add(b, b) implies a = b
} by {
    if r2_add(a, a) = r2_add(b, b) {
        r2_add(a, a).first = r2_add(b, b).first
        r2_add(a, a).first = a.first + a.first
        r2_add(b, b).first = b.first + b.first
        a.first + a.first = b.first + b.first
        real_double_eq_double(a.first, b.first)
        a.first = b.first
        r2_add(a, a).second = r2_add(b, b).second
        r2_add(a, a).second = a.second + a.second
        r2_add(b, b).second = b.second + b.second
        a.second + a.second = b.second + b.second
        real_double_eq_double(a.second, b.second)
        a.second = b.second
        a.first = b.first and a.second = b.second
        r2_pair_ext(a, b)
        a = b
    }
}

// ---------------------------------------------------------------------------
// Parallelogram predicate and four-point rearrangements, for Varignon's
// theorem.
// ---------------------------------------------------------------------------

/// A parallelogram is a quadrilateral whose diagonals share a midpoint.
define r2_parallelogram(
    a: Pair[Real, Real], b: Pair[Real, Real],
    c: Pair[Real, Real], d: Pair[Real, Real]
) -> Bool {
    r2_add(a, c) = r2_add(b, d)
}

/// The defining equation of the parallelogram predicate.
theorem r2_parallelogram_apply(
    a: Pair[Real, Real], b: Pair[Real, Real],
    c: Pair[Real, Real], d: Pair[Real, Real]
) {
    r2_parallelogram(a, b, c, d) = (r2_add(a, c) = r2_add(b, d))
}

/// A four-term real sum may be cyclically rearranged.
theorem real_add_four_rotate(a: Real, b: Real, c: Real, d: Real) {
    (a + b) + (c + d) = (b + c) + (d + a)
} by {
    (a + b) + (c + d) = a + b + c + d
    a + b + c + d = b + c + d + a
    b + c + d + a = (b + c) + (d + a)
}

/// The four vertices of a quadrilateral sum in cyclic order.
theorem r2_add_four_rearrange(
    a: Pair[Real, Real], b: Pair[Real, Real],
    c: Pair[Real, Real], d: Pair[Real, Real]
) {
    r2_add(r2_add(a, b), r2_add(c, d)) = r2_add(r2_add(b, c), r2_add(d, a))
} by {
    let lhs = r2_add(r2_add(a, b), r2_add(c, d))
    let rhs = r2_add(r2_add(b, c), r2_add(d, a))
    lhs.first = r2_add(a, b).first + r2_add(c, d).first
    r2_add(a, b).first = a.first + b.first
    r2_add(c, d).first = c.first + d.first
    lhs.first = (a.first + b.first) + (c.first + d.first)
    rhs.first = r2_add(b, c).first + r2_add(d, a).first
    r2_add(b, c).first = b.first + c.first
    r2_add(d, a).first = d.first + a.first
    rhs.first = (b.first + c.first) + (d.first + a.first)
    real_add_four_rotate(a.first, b.first, c.first, d.first)
    (a.first + b.first) + (c.first + d.first) = (b.first + c.first) + (d.first + a.first)
    lhs.first = rhs.first
    lhs.second = r2_add(a, b).second + r2_add(c, d).second
    r2_add(a, b).second = a.second + b.second
    r2_add(c, d).second = c.second + d.second
    lhs.second = (a.second + b.second) + (c.second + d.second)
    rhs.second = r2_add(b, c).second + r2_add(d, a).second
    r2_add(b, c).second = b.second + c.second
    r2_add(d, a).second = d.second + a.second
    rhs.second = (b.second + c.second) + (d.second + a.second)
    real_add_four_rotate(a.second, b.second, c.second, d.second)
    (a.second + b.second) + (c.second + d.second) = (b.second + c.second) + (d.second + a.second)
    lhs.second = rhs.second
    pair_ext(lhs, rhs)
}

// ---------------------------------------------------------------------------
// Subtraction-algebra helpers for the Euler line theorem.
// ---------------------------------------------------------------------------

/// Adding then subtracting the same point on the left cancels.
theorem r2_add_sub_left_cancel(a: Pair[Real, Real], b: Pair[Real, Real]) {
    r2_sub(r2_add(a, b), a) = b
} by {
    let lhs = r2_sub(r2_add(a, b), a)
    lhs.first = r2_add(a, b).first - a.first
    r2_add(a, b).first = a.first + b.first
    lhs.first = (a.first + b.first) - a.first
    (a.first + b.first) - a.first = b.first
    lhs.second = r2_add(a, b).second - a.second
    r2_add(a, b).second = a.second + b.second
    lhs.second = (a.second + b.second) - a.second
    (a.second + b.second) - a.second = b.second
    pair_ext(lhs, b)
}

/// Subtracting two points successively may be done in either order.
theorem r2_sub_sub_swap(x: Pair[Real, Real], y: Pair[Real, Real], z: Pair[Real, Real]) {
    r2_sub(r2_sub(x, y), z) = r2_sub(r2_sub(x, z), y)
} by {
    let lhs = r2_sub(r2_sub(x, y), z)
    let rhs = r2_sub(r2_sub(x, z), y)
    lhs.first = r2_sub(x, y).first - z.first
    r2_sub(x, y).first = x.first - y.first
    lhs.first = (x.first - y.first) - z.first
    rhs.first = r2_sub(x, z).first - y.first
    r2_sub(x, z).first = x.first - z.first
    rhs.first = (x.first - z.first) - y.first
    (x.first - y.first) - z.first = (x.first - z.first) - y.first
    lhs.first = rhs.first
    lhs.second = r2_sub(x, y).second - z.second
    r2_sub(x, y).second = x.second - y.second
    lhs.second = (x.second - y.second) - z.second
    rhs.second = r2_sub(x, z).second - y.second
    r2_sub(x, z).second = x.second - z.second
    rhs.second = (x.second - z.second) - y.second
    (x.second - y.second) - z.second = (x.second - z.second) - y.second
    lhs.second = rhs.second
    pair_ext(lhs, rhs)
}

/// Removing a common base from two real differences gives the endpoint difference.
theorem real_sub_sub_same_base(a: Real, b: Real, c: Real) {
    (c - a) - (b - a) = c - b
} by {
    (c - a) - (b - a) = (c - a) + -(b - a)
    -(b - a) = a - b
    (c - a) + -(b - a) = (c - a) + (a - b)
    (c - a) + (a - b) = c - b
}

/// Removing a common base from two differences gives the endpoint difference.
theorem r2_sub_sub_same_base(a: Pair[Real, Real], b: Pair[Real, Real], c: Pair[Real, Real]) {
    r2_sub(r2_sub(c, a), r2_sub(b, a)) = r2_sub(c, b)
} by {
    let lhs = r2_sub(r2_sub(c, a), r2_sub(b, a))
    let rhs = r2_sub(c, b)
    lhs.first = r2_sub(c, a).first - r2_sub(b, a).first
    r2_sub(c, a).first = c.first - a.first
    r2_sub(b, a).first = b.first - a.first
    lhs.first = (c.first - a.first) - (b.first - a.first)
    rhs.first = c.first - b.first
    real_sub_sub_same_base(a.first, b.first, c.first)
    (c.first - a.first) - (b.first - a.first) = c.first - b.first
    lhs.first = rhs.first
    lhs.second = r2_sub(c, a).second - r2_sub(b, a).second
    r2_sub(c, a).second = c.second - a.second
    r2_sub(b, a).second = b.second - a.second
    lhs.second = (c.second - a.second) - (b.second - a.second)
    rhs.second = c.second - b.second
    real_sub_sub_same_base(a.second, b.second, c.second)
    (c.second - a.second) - (b.second - a.second) = c.second - b.second
    lhs.second = rhs.second
    pair_ext(lhs, rhs)
}

/// Subtracting the doubled center and then the center leaves the triple
/// center subtracted: (x - 2o) - o = x - 3o.
theorem r2_triple_sub_right(x: Pair[Real, Real], o: Pair[Real, Real]) {
    r2_sub(r2_sub(x, r2_add(o, o)), o) = r2_sub(x, r2_add(o, r2_add(o, o)))
} by {
    let lhs = r2_sub(r2_sub(x, r2_add(o, o)), o)
    let rhs = r2_sub(x, r2_add(o, r2_add(o, o)))
    lhs.first = r2_sub(x, r2_add(o, o)).first - o.first
    r2_sub(x, r2_add(o, o)).first = x.first - r2_add(o, o).first
    r2_add(o, o).first = o.first + o.first
    lhs.first = (x.first - (o.first + o.first)) - o.first
    rhs.first = x.first - r2_add(o, r2_add(o, o)).first
    r2_add(o, r2_add(o, o)).first = o.first + r2_add(o, o).first
    r2_add(o, o).first = o.first + o.first
    rhs.first = x.first - (o.first + (o.first + o.first))
    (x.first - (o.first + o.first)) - o.first = x.first - (o.first + (o.first + o.first))
    lhs.first = rhs.first
    lhs.second = r2_sub(x, r2_add(o, o)).second - o.second
    r2_sub(x, r2_add(o, o)).second = x.second - r2_add(o, o).second
    r2_add(o, o).second = o.second + o.second
    lhs.second = (x.second - (o.second + o.second)) - o.second
    rhs.second = x.second - r2_add(o, r2_add(o, o)).second
    r2_add(o, r2_add(o, o)).second = o.second + r2_add(o, o).second
    r2_add(o, o).second = o.second + o.second
    rhs.second = x.second - (o.second + (o.second + o.second))
    (x.second - (o.second + o.second)) - o.second = x.second - (o.second + (o.second + o.second))
    lhs.second = rhs.second
    pair_ext(lhs, rhs)
}

/// Subtracting then adding the same point on the right cancels.
theorem r2_sub_add_right_cancel(x: Pair[Real, Real], c: Pair[Real, Real]) {
    r2_add(r2_sub(x, c), c) = x
} by {
    let lhs = r2_add(r2_sub(x, c), c)
    lhs.first = r2_sub(x, c).first + c.first
    r2_sub(x, c).first = x.first - c.first
    lhs.first = (x.first - c.first) + c.first
    (x.first - c.first) + c.first = x.first
    lhs.second = r2_sub(x, c).second + c.second
    r2_sub(x, c).second = x.second - c.second
    lhs.second = (x.second - c.second) + c.second
    (x.second - c.second) + c.second = x.second
    pair_ext(lhs, x)
}

/// A three-point sum may be cyclically rotated.
theorem r2_add_three_rotate(a: Pair[Real, Real], b: Pair[Real, Real], c: Pair[Real, Real]) {
    r2_add(r2_add(a, b), c) = r2_add(r2_add(b, c), a)
} by {
    let lhs = r2_add(r2_add(a, b), c)
    let rhs = r2_add(r2_add(b, c), a)
    lhs.first = r2_add(a, b).first + c.first
    r2_add(a, b).first = a.first + b.first
    lhs.first = (a.first + b.first) + c.first
    rhs.first = r2_add(b, c).first + a.first
    r2_add(b, c).first = b.first + c.first
    rhs.first = (b.first + c.first) + a.first
    (a.first + b.first) + c.first = (b.first + c.first) + a.first
    lhs.first = rhs.first
    lhs.second = r2_add(a, b).second + c.second
    r2_add(a, b).second = a.second + b.second
    lhs.second = (a.second + b.second) + c.second
    rhs.second = r2_add(b, c).second + a.second
    r2_add(b, c).second = b.second + c.second
    rhs.second = (b.second + c.second) + a.second
    (a.second + b.second) + c.second = (b.second + c.second) + a.second
    lhs.second = rhs.second
    pair_ext(lhs, rhs)
}

/// Subtracting a scaled point from itself leaves the complement scaled.
theorem r2_smul_complement(t: Real, a: Pair[Real, Real]) {
    r2_sub(a, r2_smul(t, a)) = r2_smul(Real.1 - t, a)
} by {
    let ta = r2_smul(t, a)
    let lhs = r2_sub(a, ta)
    let rhs = r2_smul(Real.1 - t, a)
    ta.first = t * a.first
    ta.second = t * a.second
    lhs.first = a.first - ta.first
    lhs.first = a.first - t * a.first
    rhs.first = (Real.1 - t) * a.first
    a.first - t * a.first = (Real.1 - t) * a.first
    lhs.first = rhs.first
    lhs.second = a.second - ta.second
    lhs.second = a.second - t * a.second
    rhs.second = (Real.1 - t) * a.second
    a.second - t * a.second = (Real.1 - t) * a.second
    lhs.second = rhs.second
    pair_ext(lhs, rhs)
}

/// Moving a subtracted point into the first summand: a + (b - c) = (a - c) + b.
theorem r2_add_sub_rearrange(a: Pair[Real, Real], b: Pair[Real, Real], c: Pair[Real, Real]) {
    r2_add(a, r2_sub(b, c)) = r2_add(r2_sub(a, c), b)
} by {
    let lhs = r2_add(a, r2_sub(b, c))
    let rhs = r2_add(r2_sub(a, c), b)
    lhs.first = a.first + r2_sub(b, c).first
    r2_sub(b, c).first = b.first - c.first
    lhs.first = a.first + (b.first - c.first)
    rhs.first = r2_sub(a, c).first + b.first
    r2_sub(a, c).first = a.first - c.first
    rhs.first = (a.first - c.first) + b.first
    a.first + (b.first - c.first) = a.first + (b.first + -c.first)
    a.first + (b.first + -c.first) = a.first + b.first + -c.first
    a.first + b.first + -c.first = a.first + -c.first + b.first
    a.first + -c.first + b.first = (a.first + -c.first) + b.first
    a.first - c.first + b.first = a.first + -c.first + b.first
    a.first + (b.first - c.first) = (a.first - c.first) + b.first
    lhs.first = rhs.first
    lhs.second = a.second + r2_sub(b, c).second
    r2_sub(b, c).second = b.second - c.second
    lhs.second = a.second + (b.second - c.second)
    rhs.second = r2_sub(a, c).second + b.second
    r2_sub(a, c).second = a.second - c.second
    rhs.second = (a.second - c.second) + b.second
    a.second + (b.second - c.second) = a.second + (b.second + -c.second)
    a.second + (b.second + -c.second) = a.second + b.second + -c.second
    a.second + b.second + -c.second = a.second + -c.second + b.second
    a.second + -c.second + b.second = (a.second + -c.second) + b.second
    a.second - c.second + b.second = a.second + -c.second + b.second
    a.second + (b.second - c.second) = (a.second - c.second) + b.second
    lhs.second = rhs.second
    pair_ext(lhs, rhs)
}

// ---------------------------------------------------------------------------
// The quarter turn (rotation by 90 degrees), for the Finsler-Hadwiger,
// Van Aubel and Napoleon theorems.
// ---------------------------------------------------------------------------

/// Rotation by a quarter turn counterclockwise in the real coordinate plane.
define r2_rot90(v: Pair[Real, Real]) -> Pair[Real, Real] {
    Pair.new(-v.second, v.first)
}

/// The defining equation of the quarter turn.
theorem r2_rot90_def(v: Pair[Real, Real]) {
    r2_rot90(v) = Pair.new(-v.second, v.first)
}

/// The first coordinate of a quarter turn.
theorem r2_rot90_first(v: Pair[Real, Real]) {
    r2_rot90(v).first = -v.second
} by {
    r2_rot90_def(v)
    r2_rot90(v) = Pair.new(-v.second, v.first)
    r2_rot90(v).first = Pair.new(-v.second, v.first).first
    Pair.new(-v.second, v.first).first = -v.second
    r2_rot90(v).first = -v.second
}

/// The second coordinate of a quarter turn.
theorem r2_rot90_second(v: Pair[Real, Real]) {
    r2_rot90(v).second = v.first
} by {
    r2_rot90_def(v)
    r2_rot90(v) = Pair.new(-v.second, v.first)
    r2_rot90(v).second = Pair.new(-v.second, v.first).second
    Pair.new(-v.second, v.first).second = v.first
    r2_rot90(v).second = v.first
}

/// The quarter turn distributes over point addition.
theorem r2_rot90_add(u: Pair[Real, Real], v: Pair[Real, Real]) {
    r2_rot90(r2_add(u, v)) = r2_add(r2_rot90(u), r2_rot90(v))
} by {
    let lhs = r2_rot90(r2_add(u, v))
    let rhs = r2_add(r2_rot90(u), r2_rot90(v))
    lhs.first = -r2_add(u, v).second
    r2_add(u, v).second = u.second + v.second
    lhs.first = -(u.second + v.second)
    rhs.first = r2_rot90(u).first + r2_rot90(v).first
    r2_rot90(u).first = -u.second
    r2_rot90(v).first = -v.second
    rhs.first = -u.second + -v.second
    -(u.second + v.second) = -u.second + -v.second
    lhs.first = rhs.first
    lhs.second = r2_add(u, v).first
    r2_add(u, v).first = u.first + v.first
    rhs.second = r2_rot90(u).second + r2_rot90(v).second
    r2_rot90(u).second = u.first
    r2_rot90(v).second = v.first
    rhs.second = u.first + v.first
    lhs.second = rhs.second
    pair_ext(lhs, rhs)
}

/// The quarter turn distributes over point subtraction.
theorem r2_rot90_sub(u: Pair[Real, Real], v: Pair[Real, Real]) {
    r2_rot90(r2_sub(u, v)) = r2_sub(r2_rot90(u), r2_rot90(v))
} by {
    let lhs = r2_rot90(r2_sub(u, v))
    let rhs = r2_sub(r2_rot90(u), r2_rot90(v))
    lhs.first = -r2_sub(u, v).second
    r2_sub(u, v).second = u.second - v.second
    lhs.first = -(u.second - v.second)
    rhs.first = r2_rot90(u).first - r2_rot90(v).first
    r2_rot90(u).first = -u.second
    r2_rot90(v).first = -v.second
    rhs.first = -u.second - -v.second
    -(u.second - v.second) = -u.second + v.second
    -u.second - -v.second = -u.second + v.second
    lhs.first = rhs.first
    lhs.second = r2_sub(u, v).first
    r2_sub(u, v).first = u.first - v.first
    rhs.second = r2_rot90(u).second - r2_rot90(v).second
    r2_rot90(u).second = u.first
    r2_rot90(v).second = v.first
    rhs.second = u.first - v.first
    lhs.second = rhs.second
    pair_ext(lhs, rhs)
}

/// The quarter turn commutes with scalar multiplication.
theorem r2_rot90_smul(k: Real, v: Pair[Real, Real]) {
    r2_rot90(r2_smul(k, v)) = r2_smul(k, r2_rot90(v))
} by {
    let lhs = r2_rot90(r2_smul(k, v))
    let sv = r2_smul(k, v)
    let rv = r2_rot90(v)
    let rhs = r2_smul(k, rv)
    sv.first = k * v.first
    sv.second = k * v.second
    lhs.first = -sv.second
    lhs.first = -(k * v.second)
    rv.first = -v.second
    rv.second = v.first
    rhs.first = k * rv.first
    rhs.first = k * (-v.second)
    k * (-v.second) = -(k * v.second)
    lhs.first = rhs.first
    lhs.second = sv.first
    lhs.second = k * v.first
    rhs.second = k * rv.second
    rhs.second = k * v.first
    lhs.second = rhs.second
    pair_ext(lhs, rhs)
}

/// The quarter turn preserves the squared norm.
theorem r2_rot90_norm_sq(v: Pair[Real, Real]) {
    r2_norm_sq(r2_rot90(v)) = r2_norm_sq(v)
} by {
    r2_norm_sq(r2_rot90(v)) =
        r2_rot90(v).first * r2_rot90(v).first + r2_rot90(v).second * r2_rot90(v).second
    r2_rot90(v).first = -v.second
    r2_rot90(v).second = v.first
    r2_norm_sq(r2_rot90(v)) = (-v.second) * (-v.second) + v.first * v.first
    (-v.second) * (-v.second) = v.second * v.second
    r2_norm_sq(v) = v.first * v.first + v.second * v.second
    (-v.second) * (-v.second) + v.first * v.first = v.second * v.second + v.first * v.first
    v.second * v.second + v.first * v.first = v.first * v.first + v.second * v.second
    r2_norm_sq(r2_rot90(v)) = r2_norm_sq(v)
}

/// A vector is orthogonal to its quarter turn.
theorem r2_dot_rot90_self(v: Pair[Real, Real]) {
    r2_dot(v, r2_rot90(v)) = Real.0
} by {
    r2_dot(v, r2_rot90(v)) = v.first * r2_rot90(v).first + v.second * r2_rot90(v).second
    r2_rot90(v).first = -v.second
    r2_rot90(v).second = v.first
    r2_dot(v, r2_rot90(v)) = v.first * (-v.second) + v.second * v.first
    v.first * (-v.second) = -(v.first * v.second)
    v.second * v.first = v.first * v.second
    v.first * (-v.second) + v.second * v.first = -(v.first * v.second) + v.first * v.second
    -(v.first * v.second) + v.first * v.second = Real.0
    r2_dot(v, r2_rot90(v)) = Real.0
}

/// The product of two negated reals is the product of the reals.
theorem real_neg_mul_neg(a: Real, b: Real) {
    (-a) * (-b) = a * b
} by {
    (-a) * (-b) = -(a * (-b))
    a * (-b) = -(a * b)
    -(-(a * b)) = a * b
}

/// The quarter turn preserves the dot product.
theorem r2_dot_rot90_comm(u: Pair[Real, Real], v: Pair[Real, Real]) {
    r2_dot(r2_rot90(u), r2_rot90(v)) = r2_dot(u, v)
} by {
    r2_dot(r2_rot90(u), r2_rot90(v)) =
        r2_rot90(u).first * r2_rot90(v).first + r2_rot90(u).second * r2_rot90(v).second
    r2_rot90(u).first = -u.second
    r2_rot90(u).second = u.first
    r2_rot90(v).first = -v.second
    r2_rot90(v).second = v.first
    r2_dot(r2_rot90(u), r2_rot90(v)) = (-u.second) * (-v.second) + u.first * v.first
    real_neg_mul_neg(u.second, v.second)
    (-u.second) * (-v.second) = u.second * v.second
    r2_dot(u, v) = u.first * v.first + u.second * v.second
    (-u.second) * (-v.second) + u.first * v.first = u.second * v.second + u.first * v.first
    u.second * v.second + u.first * v.first = u.first * v.first + u.second * v.second
    r2_dot(r2_rot90(u), r2_rot90(v)) = r2_dot(u, v)
}

/// Swapping the arguments of a dot product with a quarter turn negates it.
theorem r2_dot_rot90_swap(u: Pair[Real, Real], v: Pair[Real, Real]) {
    r2_dot(u, r2_rot90(v)) = -r2_dot(v, r2_rot90(u))
} by {
    r2_dot(u, r2_rot90(v)) = u.first * r2_rot90(v).first + u.second * r2_rot90(v).second
    r2_rot90(v).first = -v.second
    r2_rot90(v).second = v.first
    r2_dot(u, r2_rot90(v)) = u.first * (-v.second) + u.second * v.first
    u.first * (-v.second) = -(u.first * v.second)
    r2_dot(u, r2_rot90(v)) = -(u.first * v.second) + u.second * v.first
    r2_dot(v, r2_rot90(u)) = v.first * r2_rot90(u).first + v.second * r2_rot90(u).second
    r2_rot90(u).first = -u.second
    r2_rot90(u).second = u.first
    r2_dot(v, r2_rot90(u)) = v.first * (-u.second) + v.second * u.first
    v.first * (-u.second) = -(v.first * u.second)
    r2_dot(v, r2_rot90(u)) = -(v.first * u.second) + v.second * u.first
    -r2_dot(v, r2_rot90(u)) = v.first * u.second - v.second * u.first
    v.second * u.first = u.first * v.second
    u.second * v.first = v.first * u.second
    v.first * u.second - v.second * u.first = u.second * v.first - u.first * v.second
    -(u.first * v.second) + u.second * v.first = u.second * v.first - u.first * v.second
    -r2_dot(v, r2_rot90(u)) = -(u.first * v.second) + u.second * v.first
    r2_dot(u, r2_rot90(v)) = -r2_dot(v, r2_rot90(u))
}

/// Applying the quarter turn twice negates the vector.
theorem r2_rot90_twice(v: Pair[Real, Real]) {
    r2_rot90(r2_rot90(v)) = r2_neg(v)
} by {
    let lhs = r2_rot90(r2_rot90(v))
    lhs.first = -r2_rot90(v).second
    r2_rot90(v).second = v.first
    lhs.first = -v.first
    lhs.second = r2_rot90(v).first
    r2_rot90(v).first = -v.second
    lhs.second = -v.second
    r2_neg(v).first = -v.first
    r2_neg(v).second = -v.second
    lhs.first = r2_neg(v).first
    lhs.second = r2_neg(v).second
    pair_ext(lhs, r2_neg(v))
}

/// Adding the origin on the left leaves a point unchanged.
theorem r2_add_zero_left(a: Pair[Real, Real]) {
    r2_add(Pair.new(Real.0, Real.0), a) = a
} by {
    let lhs = r2_add(Pair.new(Real.0, Real.0), a)
    lhs.first = Pair.new(Real.0, Real.0).first + a.first
    Pair.new(Real.0, Real.0).first = Real.0
    lhs.first = Real.0 + a.first
    Real.0 + a.first = a.first
    lhs.second = Pair.new(Real.0, Real.0).second + a.second
    Pair.new(Real.0, Real.0).second = Real.0
    lhs.second = Real.0 + a.second
    Real.0 + a.second = a.second
    pair_ext(lhs, a)
}

/// Differences of differences may be regrouped:
/// (b - a) - (d - c) = (b - d) - (a - c).
theorem r2_sub_sub_sub_rearrange(
    a: Pair[Real, Real], b: Pair[Real, Real], c: Pair[Real, Real], d: Pair[Real, Real]
) {
    r2_sub(r2_sub(b, a), r2_sub(d, c)) = r2_sub(r2_sub(b, d), r2_sub(a, c))
} by {
    let lhs = r2_sub(r2_sub(b, a), r2_sub(d, c))
    let rhs = r2_sub(r2_sub(b, d), r2_sub(a, c))
    lhs.first = r2_sub(b, a).first - r2_sub(d, c).first
    r2_sub(b, a).first = b.first - a.first
    r2_sub(d, c).first = d.first - c.first
    lhs.first = (b.first - a.first) - (d.first - c.first)
    rhs.first = r2_sub(b, d).first - r2_sub(a, c).first
    r2_sub(b, d).first = b.first - d.first
    r2_sub(a, c).first = a.first - c.first
    rhs.first = (b.first - d.first) - (a.first - c.first)
    (b.first - a.first) - (d.first - c.first) = (b.first - a.first) + (c.first - d.first)
    (b.first - a.first) + (c.first - d.first) = (b.first + c.first) - (a.first + d.first)
    (b.first - d.first) - (a.first - c.first) = (b.first - d.first) + (c.first - a.first)
    (b.first - d.first) + (c.first - a.first) = (b.first + c.first) - (d.first + a.first)
    (b.first + c.first) - (a.first + d.first) = (b.first + c.first) - (d.first + a.first)
    lhs.first = rhs.first
    lhs.second = r2_sub(b, a).second - r2_sub(d, c).second
    r2_sub(b, a).second = b.second - a.second
    r2_sub(d, c).second = d.second - c.second
    lhs.second = (b.second - a.second) - (d.second - c.second)
    rhs.second = r2_sub(b, d).second - r2_sub(a, c).second
    r2_sub(b, d).second = b.second - d.second
    r2_sub(a, c).second = a.second - c.second
    rhs.second = (b.second - d.second) - (a.second - c.second)
    (b.second - a.second) - (d.second - c.second) = (b.second - a.second) + (c.second - d.second)
    (b.second - a.second) + (c.second - d.second) = (b.second + c.second) - (a.second + d.second)
    (b.second - d.second) - (a.second - c.second) = (b.second - d.second) + (c.second - a.second)
    (b.second - d.second) + (c.second - a.second) = (b.second + c.second) - (d.second + a.second)
    (b.second + c.second) - (a.second + d.second) = (b.second + c.second) - (d.second + a.second)
    lhs.second = rhs.second
    pair_ext(lhs, rhs)
}

/// A difference of differences reassembles into a difference of sums:
/// (c - b) - (a - d) = (c + d) - (a + b).
theorem r2_sub_sub_sum_rearrange(
    a: Pair[Real, Real], b: Pair[Real, Real], c: Pair[Real, Real], d: Pair[Real, Real]
) {
    r2_sub(r2_sub(c, b), r2_sub(a, d)) = r2_sub(r2_add(c, d), r2_add(a, b))
} by {
    let lhs = r2_sub(r2_sub(c, b), r2_sub(a, d))
    let rhs = r2_sub(r2_add(c, d), r2_add(a, b))
    lhs.first = r2_sub(c, b).first - r2_sub(a, d).first
    r2_sub(c, b).first = c.first - b.first
    r2_sub(a, d).first = a.first - d.first
    lhs.first = (c.first - b.first) - (a.first - d.first)
    rhs.first = r2_add(c, d).first - r2_add(a, b).first
    r2_add(c, d).first = c.first + d.first
    r2_add(a, b).first = a.first + b.first
    rhs.first = (c.first + d.first) - (a.first + b.first)
    (c.first - b.first) - (a.first - d.first) = (c.first - b.first) + (d.first - a.first)
    (c.first - b.first) + (d.first - a.first) = (c.first + d.first) - (b.first + a.first)
    (c.first + d.first) - (b.first + a.first) = (c.first + d.first) - (a.first + b.first)
    lhs.first = rhs.first
    lhs.second = r2_sub(c, b).second - r2_sub(a, d).second
    r2_sub(c, b).second = c.second - b.second
    r2_sub(a, d).second = a.second - d.second
    lhs.second = (c.second - b.second) - (a.second - d.second)
    rhs.second = r2_add(c, d).second - r2_add(a, b).second
    r2_add(c, d).second = c.second + d.second
    r2_add(a, b).second = a.second + b.second
    rhs.second = (c.second + d.second) - (a.second + b.second)
    (c.second - b.second) - (a.second - d.second) = (c.second - b.second) + (d.second - a.second)
    (c.second - b.second) + (d.second - a.second) = (c.second + d.second) - (b.second + a.second)
    (c.second + d.second) - (b.second + a.second) = (c.second + d.second) - (a.second + b.second)
    lhs.second = rhs.second
    pair_ext(lhs, rhs)
}

/// The quarter turn commutes with negation.
theorem r2_rot90_neg(v: Pair[Real, Real]) {
    r2_rot90(r2_neg(v)) = r2_neg(r2_rot90(v))
} by {
    let lhs = r2_rot90(r2_neg(v))
    let rv = r2_rot90(v)
    let rhs = r2_neg(rv)
    lhs.first = -r2_neg(v).second
    r2_neg(v).second = -v.second
    lhs.first = -(-v.second)
    rv.first = -v.second
    rv.second = v.first
    rhs.first = -rv.first
    rhs.first = -(-v.second)
    lhs.first = rhs.first
    lhs.second = r2_neg(v).first
    r2_neg(v).first = -v.first
    lhs.second = -v.first
    rhs.second = -rv.second
    rhs.second = -v.first
    lhs.second = rhs.second
    pair_ext(lhs, rhs)
}

/// A double subtraction of differences pairs across:
/// (a - b) - (c - d) = (a - c) + (d - b).
theorem r2_sub_sub_sub_pair(
    a: Pair[Real, Real], b: Pair[Real, Real], c: Pair[Real, Real], d: Pair[Real, Real]
) {
    r2_sub(r2_sub(a, b), r2_sub(c, d)) = r2_add(r2_sub(a, c), r2_sub(d, b))
} by {
    let lhs = r2_sub(r2_sub(a, b), r2_sub(c, d))
    let rhs = r2_add(r2_sub(a, c), r2_sub(d, b))
    lhs.first = r2_sub(a, b).first - r2_sub(c, d).first
    r2_sub(a, b).first = a.first - b.first
    r2_sub(c, d).first = c.first - d.first
    lhs.first = (a.first - b.first) - (c.first - d.first)
    rhs.first = r2_sub(a, c).first + r2_sub(d, b).first
    r2_sub(a, c).first = a.first - c.first
    r2_sub(d, b).first = d.first - b.first
    rhs.first = (a.first - c.first) + (d.first - b.first)
    (a.first - b.first) - (c.first - d.first) = (a.first - b.first) + -(c.first - d.first)
    real_sub_reverse_neg(c.first, d.first)
    c.first - d.first = -(d.first - c.first)
    -(c.first - d.first) = d.first - c.first
    (a.first - b.first) + -(c.first - d.first) = (a.first - b.first) + (d.first - c.first)
    real_sub_add_distrib(a.first, b.first, d.first, c.first)
    (a.first - b.first) + (d.first - c.first) = (a.first + d.first) - (b.first + c.first)
    (a.first + d.first) - (b.first + c.first) = (a.first + d.first) - (c.first + b.first)
    real_sub_add_distrib(a.first, c.first, d.first, b.first)
    (a.first - c.first) + (d.first - b.first) = (a.first + d.first) - (c.first + b.first)
    (a.first - b.first) - (c.first - d.first) = (a.first - c.first) + (d.first - b.first)
    lhs.first = rhs.first
    lhs.second = r2_sub(a, b).second - r2_sub(c, d).second
    r2_sub(a, b).second = a.second - b.second
    r2_sub(c, d).second = c.second - d.second
    lhs.second = (a.second - b.second) - (c.second - d.second)
    rhs.second = r2_sub(a, c).second + r2_sub(d, b).second
    r2_sub(a, c).second = a.second - c.second
    r2_sub(d, b).second = d.second - b.second
    rhs.second = (a.second - c.second) + (d.second - b.second)
    (a.second - b.second) - (c.second - d.second) = (a.second - b.second) + -(c.second - d.second)
    real_sub_reverse_neg(c.second, d.second)
    c.second - d.second = -(d.second - c.second)
    -(c.second - d.second) = d.second - c.second
    (a.second - b.second) + -(c.second - d.second) = (a.second - b.second) + (d.second - c.second)
    real_sub_add_distrib(a.second, b.second, d.second, c.second)
    (a.second - b.second) + (d.second - c.second) = (a.second + d.second) - (b.second + c.second)
    (a.second + d.second) - (b.second + c.second) = (a.second + d.second) - (c.second + b.second)
    real_sub_add_distrib(a.second, c.second, d.second, b.second)
    (a.second - c.second) + (d.second - b.second) = (a.second + d.second) - (c.second + b.second)
    (a.second - b.second) - (c.second - d.second) = (a.second - c.second) + (d.second - b.second)
    lhs.second = rhs.second
    pair_ext(lhs, rhs)
}

/// Negation distributes over point addition.
theorem r2_neg_add(a: Pair[Real, Real], b: Pair[Real, Real]) {
    r2_neg(r2_add(a, b)) = r2_add(r2_neg(a), r2_neg(b))
} by {
    let lhs = r2_neg(r2_add(a, b))
    let rhs = r2_add(r2_neg(a), r2_neg(b))
    lhs.first = -r2_add(a, b).first
    r2_add(a, b).first = a.first + b.first
    lhs.first = -(a.first + b.first)
    rhs.first = r2_neg(a).first + r2_neg(b).first
    r2_neg(a).first = -a.first
    r2_neg(b).first = -b.first
    rhs.first = -a.first + -b.first
    -(a.first + b.first) = -a.first + -b.first
    lhs.first = rhs.first
    lhs.second = -r2_add(a, b).second
    r2_add(a, b).second = a.second + b.second
    lhs.second = -(a.second + b.second)
    rhs.second = r2_neg(a).second + r2_neg(b).second
    r2_neg(a).second = -a.second
    r2_neg(b).second = -b.second
    rhs.second = -a.second + -b.second
    -(a.second + b.second) = -a.second + -b.second
    lhs.second = rhs.second
    pair_ext(lhs, rhs)
}

/// Double negation on points.
theorem r2_neg_neg(a: Pair[Real, Real]) {
    r2_neg(r2_neg(a)) = a
} by {
    let lhs = r2_neg(r2_neg(a))
    lhs.first = -r2_neg(a).first
    r2_neg(a).first = -a.first
    lhs.first = -(-a.first)
    lhs.second = -r2_neg(a).second
    r2_neg(a).second = -a.second
    lhs.second = -(-a.second)
    lhs.first = a.first
    lhs.second = a.second
    pair_ext(lhs, a)
}

/// Doubling a point is scalar multiplication by two.
theorem r2_smul_double(a: Pair[Real, Real]) {
    r2_smul((Real.1 + Real.1), a) = r2_add(a, a)
} by {
    let lhs = r2_smul((Real.1 + Real.1), a)
    lhs.first = (Real.1 + Real.1) * a.first
    (Real.1 + Real.1) * a.first = a.first + a.first
    lhs.second = (Real.1 + Real.1) * a.second
    (Real.1 + Real.1) * a.second = a.second + a.second
    r2_add(a, a).first = a.first + a.first
    r2_add(a, a).second = a.second + a.second
    lhs.first = r2_add(a, a).first
    lhs.second = r2_add(a, a).second
    pair_ext(lhs, r2_add(a, a))
}

/// Scalar multiplication composes: k1 (k2 v) = (k1 k2) v.
theorem r2_smul_smul(k1: Real, k2: Real, a: Pair[Real, Real]) {
    r2_smul(k1, r2_smul(k2, a)) = r2_smul(k1 * k2, a)
} by {
    let s2 = r2_smul(k2, a)
    let s1 = r2_smul(k1, s2)
    let rhs = r2_smul(k1 * k2, a)
    s1.first = k1 * s2.first
    s2.first = k2 * a.first
    s1.first = k1 * (k2 * a.first)
    s1.second = k1 * s2.second
    s2.second = k2 * a.second
    s1.second = k1 * (k2 * a.second)
    rhs.first = (k1 * k2) * a.first
    rhs.second = (k1 * k2) * a.second
    k1 * (k2 * a.first) = (k1 * k2) * a.first
    k1 * (k2 * a.second) = (k1 * k2) * a.second
    s1.first = rhs.first
    s1.second = rhs.second
    pair_ext(s1, rhs)
    r2_smul(k1, r2_smul(k2, a)) = r2_smul(k1 * k2, a)
}

/// A point plus k times itself is (1 + k) times itself.
theorem r2_smul_add_self(k: Real, a: Pair[Real, Real]) {
    r2_add(r2_smul(k, a), a) = r2_smul(Real.1 + k, a)
} by {
    let sk = r2_smul(k, a)
    let lhs = r2_add(sk, a)
    let rhs = r2_smul(Real.1 + k, a)
    lhs.first = sk.first + a.first
    sk.first = k * a.first
    lhs.first = k * a.first + a.first
    lhs.second = sk.second + a.second
    sk.second = k * a.second
    lhs.second = k * a.second + a.second
    rhs.first = (Real.1 + k) * a.first
    rhs.second = (Real.1 + k) * a.second
    k * a.first + a.first = (Real.1 + k) * a.first
    k * a.second + a.second = (Real.1 + k) * a.second
    lhs.first = rhs.first
    lhs.second = rhs.second
    pair_ext(lhs, rhs)
    r2_add(r2_smul(k, a), a) = r2_smul(Real.1 + k, a)
}

/// k times a point minus the point is (k - 1) times the point.
theorem r2_smul_sub_self(k: Real, a: Pair[Real, Real]) {
    r2_sub(r2_smul(k, a), a) = r2_smul(k - Real.1, a)
} by {
    let sk = r2_smul(k, a)
    let lhs = r2_sub(sk, a)
    let rhs = r2_smul(k - Real.1, a)
    lhs.first = sk.first - a.first
    sk.first = k * a.first
    lhs.first = k * a.first - a.first
    lhs.second = sk.second - a.second
    sk.second = k * a.second
    lhs.second = k * a.second - a.second
    rhs.first = (k - Real.1) * a.first
    rhs.second = (k - Real.1) * a.second
    k * a.first - a.first = (k - Real.1) * a.first
    k * a.second - a.second = (k - Real.1) * a.second
    lhs.first = rhs.first
    lhs.second = rhs.second
    pair_ext(lhs, rhs)
    r2_sub(r2_smul(k, a), a) = r2_smul(k - Real.1, a)
}

/// Pairing two point sums may be regrouped:
/// (a + b) + (c + d) = (a + c) + (b + d).
theorem r2_add_pair_rearrange(
    a: Pair[Real, Real], b: Pair[Real, Real], c: Pair[Real, Real], d: Pair[Real, Real]
) {
    r2_add(r2_add(a, b), r2_add(c, d)) = r2_add(r2_add(a, c), r2_add(b, d))
} by {
    let lhs = r2_add(r2_add(a, b), r2_add(c, d))
    let rhs = r2_add(r2_add(a, c), r2_add(b, d))
    lhs.first = r2_add(a, b).first + r2_add(c, d).first
    r2_add(a, b).first = a.first + b.first
    r2_add(c, d).first = c.first + d.first
    lhs.first = (a.first + b.first) + (c.first + d.first)
    rhs.first = r2_add(a, c).first + r2_add(b, d).first
    r2_add(a, c).first = a.first + c.first
    r2_add(b, d).first = b.first + d.first
    rhs.first = (a.first + c.first) + (b.first + d.first)
    real_add_pair_rearrange(a.first, b.first, c.first, d.first)
    (a.first + b.first) + (c.first + d.first) = (a.first + c.first) + (b.first + d.first)
    lhs.first = rhs.first
    lhs.second = r2_add(a, b).second + r2_add(c, d).second
    r2_add(a, b).second = a.second + b.second
    r2_add(c, d).second = c.second + d.second
    lhs.second = (a.second + b.second) + (c.second + d.second)
    rhs.second = r2_add(a, c).second + r2_add(b, d).second
    r2_add(a, c).second = a.second + c.second
    r2_add(b, d).second = b.second + d.second
    rhs.second = (a.second + c.second) + (b.second + d.second)
    real_add_pair_rearrange(a.second, b.second, c.second, d.second)
    (a.second + b.second) + (c.second + d.second) = (a.second + c.second) + (b.second + d.second)
    lhs.second = rhs.second
    pair_ext(lhs, rhs)
}

/// Adding the origin on the right leaves a point unchanged.
theorem r2_add_zero_right(a: Pair[Real, Real]) {
    r2_add(a, Pair.new(Real.0, Real.0)) = a
} by {
    let lhs = r2_add(a, Pair.new(Real.0, Real.0))
    lhs.first = a.first + Pair.new(Real.0, Real.0).first
    Pair.new(Real.0, Real.0).first = Real.0
    lhs.first = a.first + Real.0
    a.first + Real.0 = a.first
    lhs.second = a.second + Pair.new(Real.0, Real.0).second
    Pair.new(Real.0, Real.0).second = Real.0
    lhs.second = a.second + Real.0
    a.second + Real.0 = a.second
    pair_ext(lhs, a)
}

// ---------------------------------------------------------------------------
// Cancellation and regrouping helpers for the Johnson, Pompeiu and related
// circle theorems.
// ---------------------------------------------------------------------------

/// `(a + b) - a = b`: cancelling the leading summand of a sum.
theorem r2_sub_add_right_same(a: Pair[Real, Real], b: Pair[Real, Real]) {
    r2_sub(r2_add(a, b), a) = b
} by {
    r2_sub_eq_add_neg(r2_add(a, b), a)
    r2_sub(r2_add(a, b), a) = r2_add(r2_add(a, b), r2_neg(a))
    r2_add_assoc(a, b, r2_neg(a))
    r2_add(r2_add(a, b), r2_neg(a)) = r2_add(a, r2_add(b, r2_neg(a)))
    r2_add_comm(b, r2_neg(a))
    r2_add(b, r2_neg(a)) = r2_add(r2_neg(a), b)
    r2_add(r2_add(a, b), r2_neg(a)) = r2_add(a, r2_add(r2_neg(a), b))
    r2_add_assoc(a, r2_neg(a), b)
    r2_add(r2_add(a, r2_neg(a)), b) = r2_add(a, r2_add(r2_neg(a), b))
    r2_add_neg_right(a)
    r2_add(a, r2_neg(a)) = Pair.new(Real.0, Real.0)
    r2_add(r2_add(a, r2_neg(a)), b) = r2_add(Pair.new(Real.0, Real.0), b)
    r2_add_zero_left(b)
    r2_add(Pair.new(Real.0, Real.0), b) = b
    r2_add(r2_add(a, r2_neg(a)), b) = b
    r2_sub(r2_add(a, b), a) = b
}

/// `x - (x + y) = -y`: subtracting a sum from its first summand.
theorem r2_sub_add_same_left(x: Pair[Real, Real], y: Pair[Real, Real]) {
    r2_sub(x, r2_add(x, y)) = r2_neg(y)
} by {
    r2_sub_eq_add_neg(x, r2_add(x, y))
    r2_sub(x, r2_add(x, y)) = r2_add(x, r2_neg(r2_add(x, y)))
    r2_neg_add(x, y)
    r2_neg(r2_add(x, y)) = r2_add(r2_neg(x), r2_neg(y))
    r2_sub(x, r2_add(x, y)) = r2_add(x, r2_add(r2_neg(x), r2_neg(y)))
    r2_add_assoc(x, r2_neg(x), r2_neg(y))
    r2_add(r2_add(x, r2_neg(x)), r2_neg(y)) = r2_add(x, r2_add(r2_neg(x), r2_neg(y)))
    r2_add_neg_right(x)
    r2_add(x, r2_neg(x)) = Pair.new(Real.0, Real.0)
    r2_add(r2_add(x, r2_neg(x)), r2_neg(y)) = r2_add(Pair.new(Real.0, Real.0), r2_neg(y))
    r2_add_zero_left(r2_neg(y))
    r2_add(Pair.new(Real.0, Real.0), r2_neg(y)) = r2_neg(y)
    r2_add(r2_add(x, r2_neg(x)), r2_neg(y)) = r2_neg(y)
    r2_sub(x, r2_add(x, y)) = r2_neg(y)
}

/// `(p + p) - p = p`.
theorem r2_sub_double(p: Pair[Real, Real]) {
    r2_sub(r2_add(p, p), p) = p
} by {
    r2_sub_eq_add_neg(r2_add(p, p), p)
    r2_sub(r2_add(p, p), p) = r2_add(r2_add(p, p), r2_neg(p))
    r2_add_assoc(p, p, r2_neg(p))
    r2_add(r2_add(p, p), r2_neg(p)) = r2_add(p, r2_add(p, r2_neg(p)))
    r2_add_neg_right(p)
    r2_add(p, r2_neg(p)) = Pair.new(Real.0, Real.0)
    r2_add(r2_add(p, p), r2_neg(p)) = r2_add(p, Pair.new(Real.0, Real.0))
    r2_add_zero_right(p)
    r2_add(p, Pair.new(Real.0, Real.0)) = p
    r2_sub(r2_add(p, p), p) = p
}

/// `p - (a + x) = (p - a) - x`: moving a summand out of a subtraction.
theorem r2_sub_add_move_left(p: Pair[Real, Real], a: Pair[Real, Real], x: Pair[Real, Real]) {
    r2_sub(p, r2_add(a, x)) = r2_sub(r2_sub(p, a), x)
} by {
    r2_sub_eq_add_neg(p, r2_add(a, x))
    r2_sub(p, r2_add(a, x)) = r2_add(p, r2_neg(r2_add(a, x)))
    r2_neg_add(a, x)
    r2_neg(r2_add(a, x)) = r2_add(r2_neg(a), r2_neg(x))
    r2_sub(p, r2_add(a, x)) = r2_add(p, r2_add(r2_neg(a), r2_neg(x)))
    r2_add_assoc(p, r2_neg(a), r2_neg(x))
    r2_add(r2_add(p, r2_neg(a)), r2_neg(x)) = r2_add(p, r2_add(r2_neg(a), r2_neg(x)))
    r2_sub_eq_add_neg(p, a)
    r2_sub(p, a) = r2_add(p, r2_neg(a))
    r2_add(r2_sub(p, a), r2_neg(x)) = r2_add(r2_add(p, r2_neg(a)), r2_neg(x))
    r2_add(r2_sub(p, a), r2_neg(x)) = r2_add(p, r2_add(r2_neg(a), r2_neg(x)))
    r2_sub(p, r2_add(a, x)) = r2_add(r2_sub(p, a), r2_neg(x))
    r2_sub_eq_add_neg(r2_sub(p, a), x)
    r2_sub(r2_sub(p, a), x) = r2_add(r2_sub(p, a), r2_neg(x))
    r2_sub(p, r2_add(a, x)) = r2_sub(r2_sub(p, a), x)
}

/// `(a + x) - c = x - (c - a)`: moving the subtracted point into the first
/// summand.
theorem r2_sub_add_move(a: Pair[Real, Real], x: Pair[Real, Real], c: Pair[Real, Real]) {
    r2_sub(r2_add(a, x), c) = r2_sub(x, r2_sub(c, a))
} by {
    r2_sub_through(c, a, r2_add(a, x))
    r2_sub(r2_add(a, x), c) = r2_add(r2_sub(a, c), r2_sub(r2_add(a, x), a))
    r2_sub_add_right_same(a, x)
    r2_sub(r2_add(a, x), a) = x
    r2_sub(r2_add(a, x), c) = r2_add(r2_sub(a, c), x)
    r2_add_comm(r2_sub(a, c), x)
    r2_add(r2_sub(a, c), x) = r2_add(x, r2_sub(a, c))
    r2_sub(r2_add(a, x), c) = r2_add(x, r2_sub(a, c))
    r2_add_sub_rearrange(x, a, c)
    r2_add(x, r2_sub(a, c)) = r2_add(r2_sub(x, c), a)
    r2_sub(r2_add(a, x), c) = r2_add(r2_sub(x, c), a)
    r2_sub_eq_add_neg(x, r2_sub(c, a))
    r2_sub(x, r2_sub(c, a)) = r2_add(x, r2_neg(r2_sub(c, a)))
    r2_sub_reverse_neg(c, a)
    r2_sub(c, a) = r2_neg(r2_sub(a, c))
    r2_neg_neg(r2_sub(a, c))
    r2_neg(r2_neg(r2_sub(a, c))) = r2_sub(a, c)
    r2_neg(r2_sub(c, a)) = r2_sub(a, c)
    r2_add(x, r2_neg(r2_sub(c, a))) = r2_add(x, r2_sub(a, c))
    r2_sub(x, r2_sub(c, a)) = r2_add(x, r2_sub(a, c))
    r2_add_sub_rearrange(x, a, c)
    r2_add(x, r2_sub(a, c)) = r2_add(r2_sub(x, c), a)
    r2_add(r2_sub(x, c), a) = r2_sub(x, r2_sub(c, a))
    r2_sub(r2_add(a, x), c) = r2_sub(x, r2_sub(c, a))
}

/// `(1/2)(1/2)(2)(2) x = x`: clearing the half-squared factor of four.
theorem real_half_half_four(x: Real) {
    Real.one_half * Real.one_half * ((Real.1 + Real.1) * (Real.1 + Real.1)) * x = x
} by {
    Real.one_half * (Real.1 + Real.1) = Real.1
    Real.one_half * Real.one_half * ((Real.1 + Real.1) * (Real.1 + Real.1)) =
        (Real.one_half * (Real.1 + Real.1)) * (Real.one_half * (Real.1 + Real.1))
    Real.one_half * Real.one_half * ((Real.1 + Real.1) * (Real.1 + Real.1)) = Real.1
    Real.one_half * Real.one_half * ((Real.1 + Real.1) * (Real.1 + Real.1)) * x = x
}

/// `(1/2) x + (1/2) x = x`.
theorem real_half_twice(x: Real) {
    Real.one_half * x + Real.one_half * x = x
} by {
    Real.one_half * x + Real.one_half * x = (Real.one_half + Real.one_half) * x
    one_half_plus_one_half
    Real.one_half + Real.one_half = Real.1
    (Real.one_half + Real.one_half) * x = Real.1 * x
    Real.1 * x = x
    Real.one_half * x + Real.one_half * x = x
}


// ---------------------------------------------------------------------------
// Affine-combination differences, for the Menelaus theorem.
// ---------------------------------------------------------------------------

/// Dropping a pair of parentheses in a sum: `x + (y - z) = x + y - z`.
theorem r2_paren_drop(x: Real, y: Real, z: Real) {
    x + (y - z) = x + y - z
}

/// Dropping parentheses around a sum headed by a negation:
/// `x + (-a + b) = x - a + b`.
theorem r2_neg_sum_drop(x: Real, a: Real, b: Real) {
    x + (-a + b) = x - a + b
}

/// Regrouping a flat four-term difference: `x + y - z - w = (x - z) + (y - w)`.
theorem r2_reassoc(x: Real, y: Real, z: Real, w: Real) {
    x + y - z - w = (x - z) + (y - w)
}

/// The coordinate identity behind the `bc` difference: for reals `x`, `y`,
/// `w` and parameters `t1`, `t3`,
/// `(1 - t1) x + t1 y - ((1 - t3) w + t3 x) =
///      (1 - t1 - t3)(x - w) + t1 (y - w)`.
theorem r2_affine_diff_bc_coord(t1: Real, t3: Real, x: Real, y: Real, w: Real) {
    (Real.1 - t1) * x + t1 * y - ((Real.1 - t3) * w + t3 * x) =
        (Real.1 - t1 - t3) * (x - w) + t1 * (y - w)
} by {
    // both sides collapse to (1-t1-t3)x + t1y - (1-t3)w
    // LHS: collect the x terms
    (Real.1 - t1) * x - t3 * x = (Real.1 - t1 - t3) * x
    (Real.1 - t1) * x + t1 * y - (Real.1 - t3) * w - t3 * x =
        (Real.1 - t1) * x - t3 * x + t1 * y - (Real.1 - t3) * w
    (Real.1 - t1) * x - t3 * x + t1 * y - (Real.1 - t3) * w =
        (Real.1 - t1 - t3) * x + t1 * y - (Real.1 - t3) * w
    (Real.1 - t1) * x + t1 * y - (Real.1 - t3) * w - t3 * x =
        (Real.1 - t1 - t3) * x + t1 * y - (Real.1 - t3) * w
    (Real.1 - t1) * x + t1 * y - ((Real.1 - t3) * w + t3 * x) =
        (Real.1 - t1) * x + t1 * y - (Real.1 - t3) * w - t3 * x
    (Real.1 - t1) * x + t1 * y - ((Real.1 - t3) * w + t3 * x) =
        (Real.1 - t1 - t3) * x + t1 * y - (Real.1 - t3) * w
    // RHS: expand and collect the w terms
    (Real.1 - t1 - t3) * (x - w) =
        (Real.1 - t1 - t3) * x - (Real.1 - t1 - t3) * w
    t1 * (y - w) = t1 * y - t1 * w
    (Real.1 - t1 - t3) * (x - w) + t1 * (y - w) =
        ((Real.1 - t1 - t3) * x - (Real.1 - t1 - t3) * w) + t1 * (y - w)
    ((Real.1 - t1 - t3) * x - (Real.1 - t1 - t3) * w) + t1 * (y - w) =
        ((Real.1 - t1 - t3) * x - (Real.1 - t1 - t3) * w) + (t1 * y - t1 * w)
    ((Real.1 - t1 - t3) * x - (Real.1 - t1 - t3) * w) + (t1 * y - t1 * w) =
        (Real.1 - t1 - t3) * x - (Real.1 - t1 - t3) * w + (t1 * y - t1 * w)
    r2_paren_drop(
        (Real.1 - t1 - t3) * x - (Real.1 - t1 - t3) * w, t1 * y, t1 * w)
    (Real.1 - t1 - t3) * x - (Real.1 - t1 - t3) * w + (t1 * y - t1 * w) =
        (Real.1 - t1 - t3) * x - (Real.1 - t1 - t3) * w + t1 * y - t1 * w
    (Real.1 - t1 - t3) * (x - w) + t1 * (y - w) =
        (Real.1 - t1 - t3) * x - (Real.1 - t1 - t3) * w + t1 * y - t1 * w
    // (1-t1-t3) + t1 = 1 - t3
    Real.1 - t1 - t3 = Real.1 - t3 - t1
    (Real.1 - t3 - t1) + t1 = Real.1 - t3
    (Real.1 - t1 - t3) + t1 = Real.1 - t3
    (Real.1 - t1 - t3) * w + t1 * w = ((Real.1 - t1 - t3) + t1) * w
    ((Real.1 - t1 - t3) + t1) * w = (Real.1 - t3) * w
    (Real.1 - t1 - t3) * w + t1 * w = (Real.1 - t3) * w
    (Real.1 - t1 - t3) * x - (Real.1 - t1 - t3) * w + t1 * y - t1 * w =
        (Real.1 - t1 - t3) * x + t1 * y - ((Real.1 - t1 - t3) * w + t1 * w)
    (Real.1 - t1 - t3) * x + t1 * y - ((Real.1 - t1 - t3) * w + t1 * w) =
        (Real.1 - t1 - t3) * x + t1 * y - (Real.1 - t3) * w
    (Real.1 - t1 - t3) * (x - w) + t1 * (y - w) =
        (Real.1 - t1 - t3) * x + t1 * y - (Real.1 - t3) * w
    // chain the two collapses
    (Real.1 - t1) * x + t1 * y - ((Real.1 - t3) * w + t3 * x) =
        (Real.1 - t1 - t3) * (x - w) + t1 * (y - w)
}

/// The coordinate identity behind the `ca` difference: for reals `x`, `y`,
/// `w` and parameters `t2`, `t3`,
/// `(1 - t2) y + t2 w - ((1 - t3) w + t3 x) =
///      (1 - t2)(y - w) - t3 (x - w)`.
theorem r2_affine_diff_ca_coord(t2: Real, t3: Real, x: Real, y: Real, w: Real) {
    (Real.1 - t2) * y + t2 * w - ((Real.1 - t3) * w + t3 * x) =
        (Real.1 - t2) * (y - w) - t3 * (x - w)
} by {
    // LHS: drop parens, swap, reassociate
    (Real.1 - t2) * y + t2 * w - ((Real.1 - t3) * w + t3 * x) =
        (Real.1 - t2) * y + t2 * w - (Real.1 - t3) * w - t3 * x
    (Real.1 - t2) * y + t2 * w - (Real.1 - t3) * w - t3 * x =
        (Real.1 - t2) * y + t2 * w - t3 * x - (Real.1 - t3) * w
    r2_reassoc((Real.1 - t2) * y, t2 * w, t3 * x, (Real.1 - t3) * w)
    (Real.1 - t2) * y + t2 * w - t3 * x - (Real.1 - t3) * w =
        ((Real.1 - t2) * y - t3 * x) + (t2 * w - (Real.1 - t3) * w)
    // w-collect: t2w - (1-t3)w = (t2 - 1 + t3)w
    t2 * w - (Real.1 - t3) * w = (t2 - (Real.1 - t3)) * w
    -(Real.1 - t3) = -Real.1 + t3
    t2 - (Real.1 - t3) = t2 - Real.1 + t3
    (t2 - (Real.1 - t3)) * w = (t2 - Real.1 + t3) * w
    t2 * w - (Real.1 - t3) * w = (t2 - Real.1 + t3) * w
    ((Real.1 - t2) * y - t3 * x) + (t2 * w - (Real.1 - t3) * w) =
        ((Real.1 - t2) * y - t3 * x) + (t2 - Real.1 + t3) * w
    ((Real.1 - t2) * y - t3 * x) + (t2 - Real.1 + t3) * w =
        (Real.1 - t2) * y - t3 * x + (t2 - Real.1 + t3) * w
    (Real.1 - t2) * y + t2 * w - ((Real.1 - t3) * w + t3 * x) =
        (Real.1 - t2) * y - t3 * x + (t2 - Real.1 + t3) * w
    // RHS: expand, drop parens, swap, reassociate
    (Real.1 - t2) * (y - w) =
        (Real.1 - t2) * y - (Real.1 - t2) * w
    t3 * (x - w) = t3 * x - t3 * w
    (Real.1 - t2) * (y - w) - t3 * (x - w) =
        ((Real.1 - t2) * y - (Real.1 - t2) * w) - (t3 * x - t3 * w)
    ((Real.1 - t2) * y - (Real.1 - t2) * w) - (t3 * x - t3 * w) =
        (Real.1 - t2) * y - (Real.1 - t2) * w + -(t3 * x - t3 * w)
    -(t3 * x - t3 * w) = -t3 * x + t3 * w
    (Real.1 - t2) * y - (Real.1 - t2) * w + -(t3 * x - t3 * w) =
        (Real.1 - t2) * y - (Real.1 - t2) * w + (-t3 * x + t3 * w)
    r2_neg_sum_drop(
        (Real.1 - t2) * y - (Real.1 - t2) * w, t3 * x, t3 * w)
    (Real.1 - t2) * y - (Real.1 - t2) * w + (-t3 * x + t3 * w) =
        (Real.1 - t2) * y - (Real.1 - t2) * w - t3 * x + t3 * w
    (Real.1 - t2) * y - (Real.1 - t2) * w - t3 * x + t3 * w =
        (Real.1 - t2) * y - t3 * x - (Real.1 - t2) * w + t3 * w
    r2_reassoc((Real.1 - t2) * y, t3 * w, t3 * x, (Real.1 - t2) * w)
    (Real.1 - t2) * y - t3 * x - (Real.1 - t2) * w + t3 * w =
        ((Real.1 - t2) * y - t3 * x) + (t3 * w - (Real.1 - t2) * w)
    // w-collect: t3w - (1-t2)w = (t2 - 1 + t3)w
    t3 * w - (Real.1 - t2) * w = (t3 - (Real.1 - t2)) * w
    -(Real.1 - t2) = -Real.1 + t2
    t3 - (Real.1 - t2) = t3 - Real.1 + t2
    (t3 - (Real.1 - t2)) * w = (t3 - Real.1 + t2) * w
    t3 * w - (Real.1 - t2) * w = (t3 - Real.1 + t2) * w
    (t3 - Real.1 + t2) * w = (t2 - Real.1 + t3) * w
    t3 * w - (Real.1 - t2) * w = (t2 - Real.1 + t3) * w
    ((Real.1 - t2) * y - t3 * x) + (t3 * w - (Real.1 - t2) * w) =
        ((Real.1 - t2) * y - t3 * x) + (t2 - Real.1 + t3) * w
    ((Real.1 - t2) * y - t3 * x) + (t2 - Real.1 + t3) * w =
        (Real.1 - t2) * y - t3 * x + (t2 - Real.1 + t3) * w
    (Real.1 - t2) * (y - w) - t3 * (x - w) =
        (Real.1 - t2) * y - t3 * x + (t2 - Real.1 + t3) * w
    // chain
    (Real.1 - t2) * y + t2 * w - ((Real.1 - t3) * w + t3 * x) =
        (Real.1 - t2) * (y - w) - t3 * (x - w)
}

/// The difference of two affine combinations of `b, c` and `a, b` (with the
/// same total weight one) is a combination of the side vectors:
/// `[(1 - t1) b + t1 c] - [(1 - t3) a + t3 b] =
///      (1 - t1 - t3)(b - a) + t1 (c - a)`.
theorem r2_affine_diff_bc(a: Pair[Real, Real], b: Pair[Real, Real], c: Pair[Real, Real],
    t1: Real, t3: Real) {
    r2_sub(
        r2_add(r2_smul(Real.1 - t1, b), r2_smul(t1, c)),
        r2_add(r2_smul(Real.1 - t3, a), r2_smul(t3, b))
    ) =
    r2_add(
        r2_smul(Real.1 - t1 - t3, r2_sub(b, a)),
        r2_smul(t1, r2_sub(c, a))
    )
} by {
    let x1 = r2_smul(Real.1 - t1, b)
    let x2 = r2_smul(t1, c)
    let y1 = r2_smul(Real.1 - t3, a)
    let y2 = r2_smul(t3, b)
    let s1 = r2_smul(Real.1 - t1 - t3, r2_sub(b, a))
    let s2 = r2_smul(t1, r2_sub(c, a))
    // first coordinate
    x1.first = (Real.1 - t1) * b.first
    x2.first = t1 * c.first
    y1.first = (Real.1 - t3) * a.first
    y2.first = t3 * b.first
    r2_add(r2_smul(Real.1 - t1, b), r2_smul(t1, c)).first =
        x1.first + x2.first
    r2_add(r2_smul(Real.1 - t3, a), r2_smul(t3, b)).first =
        y1.first + y2.first
    r2_sub(
        r2_add(r2_smul(Real.1 - t1, b), r2_smul(t1, c)),
        r2_add(r2_smul(Real.1 - t3, a), r2_smul(t3, b))
    ).first =
        (Real.1 - t1) * b.first + t1 * c.first -
            ((Real.1 - t3) * a.first + t3 * b.first)
    s1.first = (Real.1 - t1 - t3) * r2_sub(b, a).first
    r2_sub(b, a).first = b.first - a.first
    s2.first = t1 * r2_sub(c, a).first
    r2_sub(c, a).first = c.first - a.first
    r2_add(
        r2_smul(Real.1 - t1 - t3, r2_sub(b, a)),
        r2_smul(t1, r2_sub(c, a))
    ).first =
        s1.first + s2.first
    s1.first + s2.first =
        (Real.1 - t1 - t3) * (b.first - a.first) + t1 * (c.first - a.first)
    r2_add(
        r2_smul(Real.1 - t1 - t3, r2_sub(b, a)),
        r2_smul(t1, r2_sub(c, a))
    ).first =
        (Real.1 - t1 - t3) * (b.first - a.first) + t1 * (c.first - a.first)
    r2_affine_diff_bc_coord(t1, t3, b.first, c.first, a.first)
    (Real.1 - t1) * b.first + t1 * c.first - ((Real.1 - t3) * a.first + t3 * b.first) =
        (Real.1 - t1 - t3) * (b.first - a.first) + t1 * (c.first - a.first)
    r2_sub(
        r2_add(r2_smul(Real.1 - t1, b), r2_smul(t1, c)),
        r2_add(r2_smul(Real.1 - t3, a), r2_smul(t3, b))
    ).first =
        r2_add(
            r2_smul(Real.1 - t1 - t3, r2_sub(b, a)),
            r2_smul(t1, r2_sub(c, a))
        ).first
    // second coordinate
    x1.second = (Real.1 - t1) * b.second
    x2.second = t1 * c.second
    y1.second = (Real.1 - t3) * a.second
    y2.second = t3 * b.second
    r2_add(r2_smul(Real.1 - t1, b), r2_smul(t1, c)).second =
        x1.second + x2.second
    r2_add(r2_smul(Real.1 - t3, a), r2_smul(t3, b)).second =
        y1.second + y2.second
    r2_sub(
        r2_add(r2_smul(Real.1 - t1, b), r2_smul(t1, c)),
        r2_add(r2_smul(Real.1 - t3, a), r2_smul(t3, b))
    ).second =
        (Real.1 - t1) * b.second + t1 * c.second -
            ((Real.1 - t3) * a.second + t3 * b.second)
    s1.second = (Real.1 - t1 - t3) * r2_sub(b, a).second
    r2_sub(b, a).second = b.second - a.second
    s2.second = t1 * r2_sub(c, a).second
    r2_sub(c, a).second = c.second - a.second
    r2_add(
        r2_smul(Real.1 - t1 - t3, r2_sub(b, a)),
        r2_smul(t1, r2_sub(c, a))
    ).second =
        s1.second + s2.second
    s1.second + s2.second =
        (Real.1 - t1 - t3) * (b.second - a.second) + t1 * (c.second - a.second)
    r2_add(
        r2_smul(Real.1 - t1 - t3, r2_sub(b, a)),
        r2_smul(t1, r2_sub(c, a))
    ).second =
        (Real.1 - t1 - t3) * (b.second - a.second) + t1 * (c.second - a.second)
    r2_affine_diff_bc_coord(t1, t3, b.second, c.second, a.second)
    (Real.1 - t1) * b.second + t1 * c.second - ((Real.1 - t3) * a.second + t3 * b.second) =
        (Real.1 - t1 - t3) * (b.second - a.second) + t1 * (c.second - a.second)
    r2_sub(
        r2_add(r2_smul(Real.1 - t1, b), r2_smul(t1, c)),
        r2_add(r2_smul(Real.1 - t3, a), r2_smul(t3, b))
    ).second =
        r2_add(
            r2_smul(Real.1 - t1 - t3, r2_sub(b, a)),
            r2_smul(t1, r2_sub(c, a))
        ).second
    pair_ext(
        r2_sub(
            r2_add(r2_smul(Real.1 - t1, b), r2_smul(t1, c)),
            r2_add(r2_smul(Real.1 - t3, a), r2_smul(t3, b))
        ),
        r2_add(
            r2_smul(Real.1 - t1 - t3, r2_sub(b, a)),
            r2_smul(t1, r2_sub(c, a))
        )
    )
}

/// The difference of two affine combinations of `c, a` and `a, b`:
/// `[(1 - t2) c + t2 a] - [(1 - t3) a + t3 b] =
///      (1 - t2)(c - a) - t3 (b - a)`.
theorem r2_affine_diff_ca(a: Pair[Real, Real], b: Pair[Real, Real], c: Pair[Real, Real],
    t2: Real, t3: Real) {
    r2_sub(
        r2_add(r2_smul(Real.1 - t2, c), r2_smul(t2, a)),
        r2_add(r2_smul(Real.1 - t3, a), r2_smul(t3, b))
    ) =
    r2_sub(
        r2_smul(Real.1 - t2, r2_sub(c, a)),
        r2_smul(t3, r2_sub(b, a))
    )
} by {
    let x1 = r2_smul(Real.1 - t2, c)
    let x2 = r2_smul(t2, a)
    let y1 = r2_smul(Real.1 - t3, a)
    let y2 = r2_smul(t3, b)
    let s1 = r2_smul(Real.1 - t2, r2_sub(c, a))
    let s2 = r2_smul(t3, r2_sub(b, a))
    // first coordinate
    x1.first = (Real.1 - t2) * c.first
    x2.first = t2 * a.first
    y1.first = (Real.1 - t3) * a.first
    y2.first = t3 * b.first
    r2_add(r2_smul(Real.1 - t2, c), r2_smul(t2, a)).first =
        x1.first + x2.first
    r2_add(r2_smul(Real.1 - t3, a), r2_smul(t3, b)).first =
        y1.first + y2.first
    r2_sub(
        r2_add(r2_smul(Real.1 - t2, c), r2_smul(t2, a)),
        r2_add(r2_smul(Real.1 - t3, a), r2_smul(t3, b))
    ).first =
        (Real.1 - t2) * c.first + t2 * a.first -
            ((Real.1 - t3) * a.first + t3 * b.first)
    s1.first = (Real.1 - t2) * r2_sub(c, a).first
    r2_sub(c, a).first = c.first - a.first
    s2.first = t3 * r2_sub(b, a).first
    r2_sub(b, a).first = b.first - a.first
    r2_affine_diff_ca_coord(t2, t3, b.first, c.first, a.first)
    (Real.1 - t2) * c.first + t2 * a.first - ((Real.1 - t3) * a.first + t3 * b.first) =
        (Real.1 - t2) * (c.first - a.first) - t3 * (b.first - a.first)
    r2_sub(
        r2_add(r2_smul(Real.1 - t2, c), r2_smul(t2, a)),
        r2_add(r2_smul(Real.1 - t3, a), r2_smul(t3, b))
    ).first =
        (Real.1 - t2) * (c.first - a.first) - t3 * (b.first - a.first)
    r2_sub(s1, s2).first = s1.first - s2.first
    s1.first - s2.first =
        (Real.1 - t2) * (c.first - a.first) - t3 * (b.first - a.first)
    r2_sub(
        r2_smul(Real.1 - t2, r2_sub(c, a)),
        r2_smul(t3, r2_sub(b, a))
    ).first =
        (Real.1 - t2) * (c.first - a.first) - t3 * (b.first - a.first)
    r2_sub(
        r2_add(r2_smul(Real.1 - t2, c), r2_smul(t2, a)),
        r2_add(r2_smul(Real.1 - t3, a), r2_smul(t3, b))
    ).first =
        r2_sub(
            r2_smul(Real.1 - t2, r2_sub(c, a)),
            r2_smul(t3, r2_sub(b, a))
        ).first
    // second coordinate
    x1.second = (Real.1 - t2) * c.second
    x2.second = t2 * a.second
    y1.second = (Real.1 - t3) * a.second
    y2.second = t3 * b.second
    r2_add(r2_smul(Real.1 - t2, c), r2_smul(t2, a)).second =
        x1.second + x2.second
    r2_add(r2_smul(Real.1 - t3, a), r2_smul(t3, b)).second =
        y1.second + y2.second
    r2_sub(
        r2_add(r2_smul(Real.1 - t2, c), r2_smul(t2, a)),
        r2_add(r2_smul(Real.1 - t3, a), r2_smul(t3, b))
    ).second =
        (Real.1 - t2) * c.second + t2 * a.second -
            ((Real.1 - t3) * a.second + t3 * b.second)
    s1.second = (Real.1 - t2) * r2_sub(c, a).second
    r2_sub(c, a).second = c.second - a.second
    s2.second = t3 * r2_sub(b, a).second
    r2_sub(b, a).second = b.second - a.second
    r2_affine_diff_ca_coord(t2, t3, b.second, c.second, a.second)
    (Real.1 - t2) * c.second + t2 * a.second - ((Real.1 - t3) * a.second + t3 * b.second) =
        (Real.1 - t2) * (c.second - a.second) - t3 * (b.second - a.second)
    r2_sub(
        r2_add(r2_smul(Real.1 - t2, c), r2_smul(t2, a)),
        r2_add(r2_smul(Real.1 - t3, a), r2_smul(t3, b))
    ).second =
        (Real.1 - t2) * (c.second - a.second) - t3 * (b.second - a.second)
    r2_sub(
        r2_add(r2_smul(Real.1 - t2, c), r2_smul(t2, a)),
        r2_add(r2_smul(Real.1 - t3, a), r2_smul(t3, b))
    ).second =
        r2_sub(
            r2_smul(Real.1 - t2, r2_sub(c, a)),
            r2_smul(t3, r2_sub(b, a))
        ).second
    pair_ext(
        r2_sub(
            r2_add(r2_smul(Real.1 - t2, c), r2_smul(t2, a)),
            r2_add(r2_smul(Real.1 - t3, a), r2_smul(t3, b))
        ),
        r2_sub(
            r2_smul(Real.1 - t2, r2_sub(c, a)),
            r2_smul(t3, r2_sub(b, a))
        )
    )
}
