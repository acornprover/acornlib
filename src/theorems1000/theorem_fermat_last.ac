from nat import Nat

numerals Nat

// Fermat's Last Theorem: for every exponent `n >= 3`, the equation
//     x^n + y^n = z^n
// has no solution in positive integers `x`, `y`, `z`.  Pierre de Fermat
// claimed a proof in the margin of his copy of Diophantus's "Arithmetica"
// (1637); the first complete proof, by Andrew Wiles with Richard Taylor,
// appeared in 1995 via the modularity theorem for semistable elliptic
// curves (the Taniyama-Shimura conjecture).
//
// The statement is recorded in the library's natural-number arithmetic:
// `pow` is exponentiation on `Nat` (`x.pow(n) = x^n`), and positivity of
// the variables is the lower bound `Nat.1 <= x`.
//
// Proof sketch: the case `n = 4` has an elementary proof by infinite
// descent (Fermat's own argument: a Pythagorean triple whose legs are both
// squares cannot exist, since it would give a smaller such triple).  The
// case `n = 3` follows from the arithmetic of the Eisenstein integers.
// The general case reduces, by a classical descent argument of Fermat and
// Euler, to the case of an odd prime exponent `p`, and then splits into
// two cases: the "first case" `p` does not divide `x y z`, treated by
// Sophie Germain's theorem for primes `p` with `2 p + 1` prime (see
// theorem_sophie_germain_prime.ac in this directory), and the "second
// case", treated by descent using the fact that `p | x y z`.  The full
// proof requires the modularity theorem, far beyond the current library.
//
// theorem theorems1000_fermat_last {
//     forall(n: Nat, x: Nat, y: Nat, z: Nat) {
//         Nat.3 <= n and Nat.1 <= x and Nat.1 <= y and Nat.1 <= z
//             implies not (x.pow(n) + y.pow(n) = z.pow(n))
//     }
// }
