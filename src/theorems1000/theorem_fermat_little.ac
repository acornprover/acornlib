from nat import Nat, exp_add, exp_one, mul_comm, mul_one_right, suc_sub_one, add_imp_sub,
    lt_suc, lt_trans
from number_theory.interface import rsa_pow_congr, cancel_coprime

numerals Nat

// Fermat's little theorem (Pierre de Fermat, 1640; first published proof by
// Euler in 1736): if `p` is prime, then for every natural number `a`,
//     a^p ≡ a  (mod p).
// The classical corollary, which Fermat stated, is the coprime form: if `p`
// does not divide `a`, then
//     a^(p - 1) ≡ 1  (mod p).
// The theorem is the foundation of the RSA cryptosystem (where it appears as
// the congruence `m^(k(p-1)+1) ≡ m (mod p)` for every exponent of the form
// `k(p - 1) + 1` — the library theorem `rsa_pow_congr`) and is a special
// case of Euler's theorem (`a^φ(n) ≡ 1 (mod n)` at the prime `n = p`, where
// `φ(p) = p - 1`; see theorem_euler_theorem.ac in this directory).
//
// Proof (Euler's combinatorial argument): expand `(a + b)^p` by the binomial
// theorem.  Every middle binomial coefficient `binom(p, k)`, `1 <= k <= p-1`,
// is divisible by the prime `p` (since `p` divides `p!` but not `k!(p-k)!`),
// so the expansion collapses to `(a + b)^p ≡ a^p + b^p (mod p)` — the
// "freshman's dream".  Iterating gives `a^p ≡ a (mod p)`; when `p` does not
// divide `a`, cancelling the factor `a` (coprime to `p`) yields
// `a^(p - 1) ≡ 1 (mod p)`.
//
// The proofs below realize this via the library's `rsa_pow_congr` (which
// formalizes the freshman's-dream induction and, at `k = 1`, states
// `a^p ≡ a (mod p)` directly) and modular cancellation `cancel_coprime`.

/// For a prime `p`, the exponent `(p - 1) + 1` collapses to `p`.
theorem fermat_pred_plus_one(p: Nat) {
    p.is_prime implies (p - Nat.1) + Nat.1 = p
} by {
    if p.is_prime {
        Nat.1 < p
        lt_suc(Nat.0)
        Nat.0 < Nat.1
        lt_trans(Nat.0, Nat.1, p)
        Nat.0 < p
        let q: Nat satisfy { q.suc = p }
        q.suc = p
        suc_sub_one(q)
        q.suc - Nat.1 = q
        p - Nat.1 = q
        q + Nat.1 = q.suc
        q + Nat.1 = p
        (p - Nat.1) + Nat.1 = p
    }
}

/// Fermat's little theorem: `a^p ≡ a (mod p)` for prime `p`.
theorem theorems1000_fermat_little(p: Nat, a: Nat) {
    p.is_prime implies a.pow(p).congr_mod(a, p)
} by {
    if p.is_prime {
        rsa_pow_congr(p, a, Nat.1)
        a.pow(Nat.1 * (p - Nat.1) + Nat.1).congr_mod(a, p)
        Nat.1 * (p - Nat.1) = p - Nat.1
        Nat.1 * (p - Nat.1) + Nat.1 = (p - Nat.1) + Nat.1
        fermat_pred_plus_one(p)
        (p - Nat.1) + Nat.1 = p
        Nat.1 * (p - Nat.1) + Nat.1 = p
        a.pow(p).congr_mod(a, p)
    }
}

/// The classical corollary: `a^(p - 1) ≡ 1 (mod p)` whenever `p` is prime
/// and does not divide `a`.
theorem theorems1000_fermat_little_corollary(p: Nat, a: Nat) {
    p.is_prime and a.coprime(p) implies a.pow(p - Nat.1).congr_mod(Nat.1, p)
} by {
    if p.is_prime and a.coprime(p) {
        // a^p ≡ a (mod p), and a^p = a^(p-1) * a.
        theorems1000_fermat_little(p, a)
        a.pow(p).congr_mod(a, p)
        exp_add(a, p - Nat.1, Nat.1)
        a.pow((p - Nat.1) + Nat.1) = a.pow(p - Nat.1) * a.pow(Nat.1)
        exp_one(a)
        a.pow(Nat.1) = a
        fermat_pred_plus_one(p)
        (p - Nat.1) + Nat.1 = p
        a.pow(p) = a.pow(p - Nat.1) * a
        (a.pow(p - Nat.1) * a).congr_mod(a, p)
        // Rewrite the right side as a * 1 and cancel the factor a, which is
        // coprime to p.
        mul_one_right(a)
        a * Nat.1 = a
        (a.pow(p - Nat.1) * a).congr_mod(a * Nat.1, p)
        mul_comm(a.pow(p - Nat.1), a)
        a.pow(p - Nat.1) * a = a * a.pow(p - Nat.1)
        (a * a.pow(p - Nat.1)).congr_mod(a * Nat.1, p)
        cancel_coprime(a, p, a.pow(p - Nat.1), Nat.1)
        a.pow(p - Nat.1).congr_mod(Nat.1, p)
    }
}
