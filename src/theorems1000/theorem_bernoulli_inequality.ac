from nat import Nat, from_nat, from_nat_zero, from_nat_add, from_nat_one, pow_zero,
    pow_one, pow_add
from real import Real, lte_trans, lte_add_left, lte_add_right, add_lte_add, mul_nonneg,
    square_nonneg, mul_zero_left, add_zero_right, lte_mul_nonneg_right,
    zero_lte_imp_non_neg, from_nat_suc_pos_real
from order import lte_ref, lte_of_lt

numerals Nat

// Bernoulli's inequality (Jacob Bernoulli, 1689): for every natural `n` and
// every real `x >= -1`,
//     (1 + x)^n >= 1 + n x.
// The inequality is the basic linear lower bound for powers near one: it
// shows that `(1 + x)^n` grows at least linearly in `n` for fixed `x > 0`
// (so `(1 + 1/n)^n` stays bounded below by 2), it underlies the proof that
// `lim (1 + 1/n)^n = e` converges, and it is used in many real-analysis
// estimates (e.g. to show `n^(1/n) -> 1`).  The restriction `x >= -1` is
// necessary: the right-hand side would outgrow the left for `x < -1` at
// even `n` (e.g. `x = -2`, `n = 2`: `(-1)^2 = 1 < 1 - 4 = -3` is fine, but
// `n = 3`: `(-1)^3 = -1 < 1 - 6`).
//
// Proof by induction on `n`.  The case `n = 0` is `1 >= 1`.  For the step,
// assume `(1 + x)^n >= 1 + n x` and multiply both sides by the nonnegative
// factor `1 + x` (here `x >= -1` enters):
//     (1 + x)^(n+1) >= (1 + n x)(1 + x) = 1 + (n + 1) x + n x^2 >= 1 + (n+1) x,
// where the last inequality uses `n x^2 >= 0`.  The lemmas below assemble
// exactly these steps from the real ordered-field interface.

/// The factor `1 + x` is nonnegative when `x >= -1`.
theorem bernoulli_one_plus_x_nonneg(x: Real) {
    -Real.1 <= x implies Real.0 <= Real.1 + x
} by {
    if -Real.1 <= x {
        lte_add_left(-Real.1, x, Real.1)
        Real.1 + -Real.1 <= Real.1 + x
        Real.1 + -Real.1 = Real.0
        Real.0 <= Real.1 + x
    }
}

/// The real image of a natural number is nonnegative.
theorem from_nat_nonneg_real(k: Nat) {
    from_nat[Real](k) >= Real.0
} by {
    define p(j: Nat) -> Bool {
        from_nat[Real](j) >= Real.0
    }
    from_nat_zero[Real]
    from_nat[Real](Nat.0) = Real.0
    lte_ref(Real.0)
    Real.0 <= Real.0
    p(Nat.0)
    forall(j: Nat) {
        if p(j) {
            from_nat_suc_pos_real(j)
            from_nat[Real](j.suc) > Real.0
            Real.0 < from_nat[Real](j.suc)
            lte_of_lt[Real](Real.0, from_nat[Real](j.suc))
            from_nat[Real](j.suc) >= Real.0
            p(j.suc)
        }
    }
    p(Nat.0) and forall(j: Nat) { p(j) implies p(j.suc) }
    Nat.induction(p)
    p(k)
}

/// The correction term `n x^2` of the induction step is nonnegative.
theorem bernoulli_sq_term_nonneg(x: Real, k: Nat) {
    from_nat[Real](k) * (x * x) >= Real.0
} by {
    from_nat_nonneg_real(k)
    from_nat[Real](k) >= Real.0
    square_nonneg(x)
    x * x >= Real.0
    mul_nonneg(from_nat[Real](k), x * x)
    from_nat[Real](k) * (x * x) >= Real.0
}

/// The successor step of from_nat in a product: `(n+1) x = n x + x`.
theorem from_nat_suc_mul(x: Real, k: Nat) {
    from_nat[Real](k.suc) * x = from_nat[Real](k) * x + x
} by {
    from_nat_add[Real](k, Nat.1)
    from_nat[Real](k + Nat.1) = from_nat[Real](k) + from_nat[Real](Nat.1)
    k + Nat.1 = k.suc
    from_nat[Real](k.suc) = from_nat[Real](k) + from_nat[Real](Nat.1)
    from_nat_one[Real]
    from_nat[Real](Nat.1) = Real.1
    from_nat[Real](k.suc) = from_nat[Real](k) + Real.1
    from_nat[Real](k.suc) * x = from_nat[Real](k) * x + x
}

/// The expansion of the step: `(1 + n x)(1 + x) = 1 + (n+1) x + n x^2`.
theorem bernoulli_expand(x: Real, k: Nat) {
    (Real.1 + from_nat[Real](k) * x) * (Real.1 + x) =
        Real.1 + from_nat[Real](k.suc) * x + from_nat[Real](k) * (x * x)
} by {
    from_nat_suc_mul(x, k)
    from_nat[Real](k.suc) * x = from_nat[Real](k) * x + x
    (Real.1 + from_nat[Real](k) * x) * (Real.1 + x) =
        (Real.1 + from_nat[Real](k) * x) * Real.1 +
        (Real.1 + from_nat[Real](k) * x) * x
    (Real.1 + from_nat[Real](k) * x) * Real.1 = Real.1 + from_nat[Real](k) * x
    (Real.1 + from_nat[Real](k) * x) * x = Real.1 * x + (from_nat[Real](k) * x) * x
    Real.1 * x = x
    (from_nat[Real](k) * x) * x = from_nat[Real](k) * (x * x)
    (Real.1 + from_nat[Real](k) * x) * x = x + from_nat[Real](k) * (x * x)
    (Real.1 + from_nat[Real](k) * x) * (Real.1 + x) =
        (Real.1 + from_nat[Real](k) * x) + (x + from_nat[Real](k) * (x * x))
    (Real.1 + from_nat[Real](k) * x) + (x + from_nat[Real](k) * (x * x)) =
        Real.1 + from_nat[Real](k) * x + x + from_nat[Real](k) * (x * x)
    (Real.1 + from_nat[Real](k) * x) * (Real.1 + x) =
        Real.1 + from_nat[Real](k) * x + x + from_nat[Real](k) * (x * x)
    Real.1 + from_nat[Real](k) * x + x + from_nat[Real](k) * (x * x) =
        Real.1 + from_nat[Real](k.suc) * x + from_nat[Real](k) * (x * x)
    (Real.1 + from_nat[Real](k) * x) * (Real.1 + x) =
        Real.1 + from_nat[Real](k.suc) * x + from_nat[Real](k) * (x * x)
}

/// The induction step of Bernoulli's inequality.
theorem bernoulli_step(x: Real, k: Nat) {
    -Real.1 <= x and
    (Real.1 + x).pow(k) >= Real.1 + from_nat[Real](k) * x
        implies (Real.1 + x).pow(k.suc) >= Real.1 + from_nat[Real](k.suc) * x
} by {
    if -Real.1 <= x and (Real.1 + x).pow(k) >= Real.1 + from_nat[Real](k) * x {
        // (1 + x)^(k+1) = (1 + x)^k * (1 + x).
        pow_add[Real](Real.1 + x, k, Nat.1)
        (Real.1 + x).pow(k + Nat.1) = (Real.1 + x).pow(k) * (Real.1 + x).pow(Nat.1)
        k + Nat.1 = k.suc
        (Real.1 + x).pow(k.suc) = (Real.1 + x).pow(k) * (Real.1 + x).pow(Nat.1)
        pow_one[Real](Real.1 + x)
        (Real.1 + x).pow(Nat.1) = Real.1 + x
        (Real.1 + x).pow(k.suc) = (Real.1 + x).pow(k) * (Real.1 + x)
        // Multiply the induction hypothesis by the nonnegative 1 + x.
        bernoulli_one_plus_x_nonneg(x)
        Real.0 <= Real.1 + x
        zero_lte_imp_non_neg(Real.1 + x)
        not (Real.1 + x).is_negative
        lte_mul_nonneg_right(Real.1 + from_nat[Real](k) * x, (Real.1 + x).pow(k), Real.1 + x)
        (Real.1 + from_nat[Real](k) * x) * (Real.1 + x) <= (Real.1 + x).pow(k) * (Real.1 + x)
        (Real.1 + from_nat[Real](k) * x) * (Real.1 + x) <= (Real.1 + x).pow(k.suc)
        // Expand the left side.
        bernoulli_expand(x, k)
        (Real.1 + from_nat[Real](k) * x) * (Real.1 + x) =
            Real.1 + from_nat[Real](k.suc) * x + from_nat[Real](k) * (x * x)
        Real.1 + from_nat[Real](k.suc) * x + from_nat[Real](k) * (x * x) <= (Real.1 + x).pow(k.suc)
        // The correction term n x^2 is nonnegative, so 1 + (n+1)x is the
        // smaller side.
        bernoulli_sq_term_nonneg(x, k)
        from_nat[Real](k) * (x * x) >= Real.0
        lte_add_left(Real.0, from_nat[Real](k) * (x * x), Real.1 + from_nat[Real](k.suc) * x)
        Real.1 + from_nat[Real](k.suc) * x + Real.0 <= Real.1 + from_nat[Real](k.suc) * x + from_nat[Real](k) * (x * x)
        add_zero_right(Real.1 + from_nat[Real](k.suc) * x)
        Real.1 + from_nat[Real](k.suc) * x + Real.0 = Real.1 + from_nat[Real](k.suc) * x
        Real.1 + from_nat[Real](k.suc) * x <= Real.1 + from_nat[Real](k.suc) * x + from_nat[Real](k) * (x * x)
        lte_trans(Real.1 + from_nat[Real](k.suc) * x,
            Real.1 + from_nat[Real](k.suc) * x + from_nat[Real](k) * (x * x),
            (Real.1 + x).pow(k.suc))
        Real.1 + from_nat[Real](k.suc) * x <= (Real.1 + x).pow(k.suc)
        (Real.1 + x).pow(k.suc) >= Real.1 + from_nat[Real](k.suc) * x
    }
}

/// Bernoulli's inequality: `(1 + x)^n >= 1 + n x` for `x >= -1`.
theorem theorems1000_bernoulli_inequality(x: Real, n: Nat) {
    -Real.1 <= x implies (Real.1 + x).pow(n) >= Real.1 + from_nat[Real](n) * x
} by {
    if -Real.1 <= x {
        define p(k: Nat) -> Bool {
            (Real.1 + x).pow(k) >= Real.1 + from_nat[Real](k) * x
        }
        pow_zero[Real](Real.1 + x)
        (Real.1 + x).pow(Nat.0) = Real.1
        from_nat_zero[Real]
        from_nat[Real](Nat.0) = Real.0
        mul_zero_left(x)
        Real.0 * x = Real.0
        from_nat[Real](Nat.0) * x = Real.0
        add_zero_right(Real.1)
        Real.1 + Real.0 = Real.1
        Real.1 + from_nat[Real](Nat.0) * x = Real.1
        lte_ref(Real.1)
        Real.1 <= Real.1
        (Real.1 + x).pow(Nat.0) = Real.1 + from_nat[Real](Nat.0) * x
        (Real.1 + x).pow(Nat.0) >= Real.1 + from_nat[Real](Nat.0) * x
        p(Nat.0)
        forall(k: Nat) {
            if p(k) {
                bernoulli_step(x, k)
                (Real.1 + x).pow(k.suc) >= Real.1 + from_nat[Real](k.suc) * x
                p(k.suc)
            }
            p(k) implies p(k.suc)
        }
        p(Nat.0) and forall(k: Nat) { p(k) implies p(k.suc) }
        Nat.induction(p)
        p(n)
        (Real.1 + x).pow(n) >= Real.1 + from_nat[Real](n) * x
    }
}
