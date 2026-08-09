// The central limit theorem (Pierre-Simon Laplace, 1810; the name is due
// to George Pólya, 1920; the modern formulation to Aleksandr Lyapunov and
// Jarl Waldemar Lindeberg): if `X_1, X_2, ...` are independent, identically
// distributed real random variables with finite mean `μ` and finite
// positive variance `σ²`, then the standardized partial sums converge in
// distribution to the standard normal law: for every real `t`,
//     P( (S_n - n μ) / (σ √n) ≤ t ) → Φ(t)   as n → infinity,
// where `S_n = X_1 + ... + X_n` and
//     Φ(t) = (1 / √(2 π)) ∫_{-infinity}^{t} exp(-u² / 2) du
// is the cumulative distribution function of the standard normal
// distribution.  In words: the sum of many small independent random
// contributions is approximately normal, whatever the common law of the
// summands (as long as it has a finite second moment).  This "universality"
// is why the normal distribution dominates statistics and the theory of
// measurement error; the theorem also explains the de Moivre–Laplace
// approximation of the binomial distribution (the "Theorem of de Moivre–
// Laplace" entry of the "1000+ theorems" list) as the special case of
// Bernoulli summands.
//
// The classical proof (Lévy, Lindeberg) uses characteristic functions:
// `φ_{S_n}(t) = φ(t/n)^{n}` for the standardized sum, the Taylor expansion
// of the characteristic function `φ` of a summand up to second order at
// zero, and the Lévy continuity theorem, which upgrades the pointwise
// convergence of characteristic functions to convergence in distribution.
// The Lindeberg–Lévy theorem gives sufficient conditions on the summands;
// Lyapunov's version (an entry of its own on the list) assumes the
// existence of a moment of order `2 + δ`.
//
// The library has no probability theory (no probability spaces, random
// variables, or distribution functions), so a faithful formal statement
// cannot even be typed; the statement is therefore recorded as prose with
// a schematic skeleton.  The convergence-in-distribution conclusion would
// be stated, in the library's sequence notation, as the pointwise
// convergence of the cumulative distribution functions of the standardized
// sums to the normal distribution function `Φ`.

// theorem theorems1000_central_limit {
//     // Schematic: for i.i.d. summands with mean mu and variance sigma^2,
//     // the cdf of (S_n - n mu) / (sigma sqrt(n)) converges pointwise to
//     // the standard normal cdf Phi.
//     //   converges_to(function(n: Nat) { cdf_standardized_sum(n, t) }, Phi(t))
// }
