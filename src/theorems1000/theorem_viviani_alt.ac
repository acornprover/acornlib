from pair import Pair
from real import Real
from top100 import r2_add, r2_sub, r2_norm_sq
from theorems1000.r2_helpers import r2_cross

// Viviani's theorem: in an equilateral triangle, the sum of the distances
// from any point to the three sides equals the altitude of the triangle.
//
// The equilateral triangle has vertices `a`, `b`, `c` and side length `s`
// (witnessed by `s^2 = |b-a|^2 = |c-b|^2 = |a-c|^2`), and its altitude `h`
// is the distance from `a` to the midpoint `m` of `bc`
// (`h^2 = |a-m|^2`, `b + c = 2m`).  The distances `d1`, `d2`, `d3` from the
// point `p` to the sides `ab`, `bc`, `ca` are witnessed through the squared
// distance to a line: `dist(p, line(u, v))^2 = cross(p - u, v - u)^2 / |v-u|^2`,
// so `d1^2 s^2 = cross(p - a, b - a)^2` and cyclically.
//
// The statement is commented out because the proof needs the signed doubled
// area (the cross product), the split of the doubled area of the big
// triangle into the three doubled areas of the subtriangles with `p`, and
// the identification `|cross(v - u, p - u)| = |v - u| * dist(p, line(u, v))`
// in squared form; each of these is a chain of cross-product and absolute
// value steps.  The sketch is recorded below.
//
// Proof sketch: doubled areas add, so with side `s` and the distances `d1`,
// `d2`, `d3` from `p` to the three sides,
//     cross(b - a, c - a) = cross(b - a, p - a) + cross(c - b, p - b) + cross(a - c, p - c),
// and each of the three summands on the right has absolute value `s * di`
// (the base `s` times the doubled height).  For an interior point `p` all
// three crosses have the same orientation, so the absolute values add:
//     s * (d1 + d2 + d3) = |cross(b - a, c - a)| = s * h,
// and cancelling the nonzero `s` gives `d1 + d2 + d3 = h`.
//
// theorem theorems1000_viviani(
//     a: Pair[Real, Real], b: Pair[Real, Real], c: Pair[Real, Real],
//     m: Pair[Real, Real], p: Pair[Real, Real],
//     s: Real, h: Real, d1: Real, d2: Real, d3: Real
// ) {
//     s * s = r2_norm_sq(r2_sub(b, a)) and
//     s * s = r2_norm_sq(r2_sub(c, b)) and
//     s * s = r2_norm_sq(r2_sub(a, c)) and
//     r2_add(m, m) = r2_add(b, c) and
//     h * h = r2_norm_sq(r2_sub(a, m)) and
//     d1 * d1 * (s * s) = r2_cross(r2_sub(p, a), r2_sub(b, a)) * r2_cross(r2_sub(p, a), r2_sub(b, a)) and
//     d2 * d2 * (s * s) = r2_cross(r2_sub(p, b), r2_sub(c, b)) * r2_cross(r2_sub(p, b), r2_sub(c, b)) and
//     d3 * d3 * (s * s) = r2_cross(r2_sub(p, c), r2_sub(a, c)) * r2_cross(r2_sub(p, c), r2_sub(a, c))
//     implies
//     d1 + d2 + d3 = h
// }
