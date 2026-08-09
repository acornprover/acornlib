from pair import Pair
from real import Real
from top100 import r2_add, r2_sub, r2_norm_sq
from theorems1000.r2_helpers import r2_smul

// The butterfly theorem: let `m` be the midpoint of a chord `pq` of a
// circle.  Through `m` draw two further chords `ab` and `cd`.  If the lines
// `ad` and `bc` meet the line `pq` at `x` and `y` respectively, then `m` is
// the midpoint of the segment `xy`: `x + y = 2m`.  (The name comes from the
// resemblance of the figure to a butterfly.)
//
// The statement uses witnesses throughout: `p`, `q`, `a`, `b`, `c`, `d` lie
// on the circle centered at `o` with squared radius `radius_sq`; `m` is the
// midpoint of `pq` (`p + q = 2m`); the chords `ab` and `cd` pass through `m`
// (affine parameters `s1`, `t1`); the intersection points `x` on `ad` and
// `pq` and `y` on `bc` and `pq` are witnessed by their two line parameters
// each.  The conclusion is the midpoint identity `x + y = 2m`.
//
// Proof sketch (using the power of a point twice): by the intersecting-chords
// theorem at `x` on the secants `xad` and `xpq`, the products of the signed
// distances satisfy `xa * xd = xp * xq`; at `y`, `yb * yc = yp * yq`.  The
// chords through `m` give `ma * mb = mc * md` and `mp * mq = ... = - (pq/2)^2`
// in signed form.  Writing `x = m + r (q - p)`-style parameters along `pq`
// and using the chord product at `m` to relate the four parameters on the
// two chords through `m`, one obtains the two intersection parameters `r`
// and `r'` and shows `r + r' = 0`, i.e. `x + y = 2m`.  In coordinates the
// prover-visible proof would expand the four circle memberships
// (`|p-o|^2 = ... = |d-o|^2 = radius_sq`) with the norm-square expansion and
// the scalar laws, solve the two pairs of line equations for the witnesses,
// and finish with the chord-power identity from `point2_chord`; the
// statement is recorded here with all witnesses explicit.
//
// theorem theorems1000_butterfly(
//     o: Pair[Real, Real], radius_sq: Real,
//     p: Pair[Real, Real], q: Pair[Real, Real], m: Pair[Real, Real],
//     a: Pair[Real, Real], b: Pair[Real, Real],
//     c: Pair[Real, Real], d: Pair[Real, Real],
//     x: Pair[Real, Real], y: Pair[Real, Real],
//     s1: Real, t1: Real, u1: Real, u2: Real, v1: Real, v2: Real
// ) {
//     r2_norm_sq(r2_sub(p, o)) = radius_sq and
//     r2_norm_sq(r2_sub(q, o)) = radius_sq and
//     r2_norm_sq(r2_sub(a, o)) = radius_sq and
//     r2_norm_sq(r2_sub(b, o)) = radius_sq and
//     r2_norm_sq(r2_sub(c, o)) = radius_sq and
//     r2_norm_sq(r2_sub(d, o)) = radius_sq and
//     r2_add(p, q) = r2_add(m, m) and
//     r2_add(a, r2_smul(s1, r2_sub(b, a))) = m and
//     r2_add(c, r2_smul(t1, r2_sub(d, c))) = m and
//     r2_add(a, r2_smul(u1, r2_sub(d, a))) = x and
//     r2_add(p, r2_smul(u2, r2_sub(q, p))) = x and
//     r2_add(b, r2_smul(v1, r2_sub(c, b))) = y and
//     r2_add(p, r2_smul(v2, r2_sub(q, p))) = y
//     implies
//     r2_add(x, y) = r2_add(m, m)
// }
