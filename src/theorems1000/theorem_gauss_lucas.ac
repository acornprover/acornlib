from pair import Pair
from real import Real
from list import partial
from theorems1000.r2_helpers import r2_smul

// The Gauss–Lucas theorem: the critical points of a nonconstant complex
// polynomial all lie in the convex hull of its roots.
//
// The plane is represented by real coordinate pairs, and complex numbers
// are the same pairs; complex conjugation is the reflection
// `conj(x, y) = (x, -y)`.  A polynomial with roots `z(0), ..., z(n-1)` has
// derivative condition `P'(w) = 0`.  For a critical point `w` that is not a
// root,
//     P'(w) / P(w) = sum_i 1 / (w - z(i)) = 0,
// and multiplying by the conjugate of the nonzero product of the
// displacements gives the conjugate-weighted identity
//     sum_i conj(w - z(i)) * prod_{j != i} |w - z(j)|^2 = 0,
// i.e., with the nonnegative weights
//     weight(i) = prod_{j != i} |w - z(j)|^2 >= 0,
//     (sum weight) * conj(w) = sum weight(i) * conj(z(i)).
// Dividing by the positive total shows that `conj(w)` is a convex
// combination of the points `conj(z(i))`; conjugating back (conjugation is
// an involution and the weights are real), `w` itself is a convex
// combination of the roots `z(i)` — the content of the Gauss–Lucas theorem.
//
// The statement below records the normalized conclusion: given nonnegative
// weights `weight` with total `total != 0` satisfying the conjugate-weighted
// identity above (which is exactly the critical-point condition), there are
// nonnegative coefficients `l` with `sum l = 1` and `w = sum l(i) z(i)` —
// `w` lies in the convex hull of the roots.  The proof takes
// `l(i) = weight(i) / total`: the division, the positivity arithmetic, and
// the conjugation back require the real field operations on coordinates and
// the scalar-vector algebra, which are available but make the proof a long
// chain; the statement is therefore recorded with the derivation sketched
// above.
//
// theorem theorems1000_gauss_lucas(
//     z: Nat -> Pair[Real, Real], n: Nat, w: Pair[Real, Real],
//     weight: Nat -> Real, total: Real
// ) {
//     forall(i: Nat) { i < n implies Real.0 <= weight(i) } and
//     partial[Real](weight, n) = total and
//     total != Real.0 and
//     r2_smul(total, conj2(w)) =
//         weighted_sum(weight, function(i: Nat) -> Pair[Real, Real] { conj2(z(i)) }, n)
//     implies
//     exists(l: Nat -> Real) {
//         forall(i: Nat) { i < n implies Real.0 <= l(i) } and
//         partial[Real](l, n) = Real.1 and
//         w = weighted_sum(l, z, n)
//     }
// }
