from nat import Nat

// Lucas's theorem: for a prime `p`, the binomial coefficient `binom(n, k)`
// modulo `p` is the product, over the base-`p` digits, of the binomial
// coefficients of the individual digits:
//     binom(n, k) == product_i binom(n_i, k_i)  (mod p),
// where `n_i` and `k_i` are the digits of `n` and `k` in base `p`.  In
// symbols, with `lucas_digit(p, i, x) = (x div p^i) mod p` and the product
// ranging over `i = 0, ..., n` (digits beyond the length of `n` contribute
// `binom(0, 0) = 1` because `k <= n`), the statement is
//     binom(n, k)  congr_mod  range_prod(...)  p.
//
// The proof (sketched below) is by strong induction on `n` using the base-p
// division `n = n0 + p q`, the recurrence for `binom`, and the congruence
// `binom(n, k) congr_mod binom(n0, k0) * binom(q, k div p)` obtained from
// the factorial form of `binom`; it is commented out because the digit
// extraction and the congruence arithmetic require a long chain of modular
// steps similar to `kummer.ac`.
//
// Proof sketch: for `n = n0 + p q` with `n0 = n mod p`, `k = k0 + p r` with
// `k0 = k mod p`:
//   1. `binom(n, k) = binom(n, k0) * (p q + (n0 - k0)) / (p r + ...)` reduces
//      to `binom(n, k) congr_mod binom(n0, k0) * binom(q, r)` (mod p) because
//      every factor of `p` in the quotient cancels;
//   2. the induction hypothesis applies to `binom(q, r)` since `q < n`;
//   3. multiplying the congruences gives the digit product.

/// The i-th base-`p` digit of `n`.
define lucas_digit(p: Nat, i: Nat, n: Nat) -> Nat {
    n.div(p.pow(i)).mod(p)
}

// Lucas's theorem: binom(n, k) is congruent modulo the prime p to the
// product of the binomial coefficients of the base-p digits of n and k.
// The product ranges over the first n + 1 digits; digits beyond the length
// of n contribute binom(0, 0) = 1, so the product is unchanged.
//
// theorem theorems1000_lucas(p: Nat, n: Nat, k: Nat) {
//     p.is_prime and k <= n implies
//     binom(n, k).congr_mod(
//         range_prod(function(i: Nat) { binom(lucas_digit(p, i, n), lucas_digit(p, i, k)) }, n.suc),
//         p)
// }
