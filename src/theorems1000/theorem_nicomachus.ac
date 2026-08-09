from list import partial, partial_zero, partial_split_last
from nat import Nat
numerals Nat

// Nicomachus's theorem: the sum of the first `n` cubes equals the square
// of the sum of the first `n` natural numbers:
//     1^3 + 2^3 + ... + n^3 = (1 + 2 + ... + n)^2.
// (An entry of the "1000+ theorems" list.)

/// The (k+1)-th cube, `(k+1)^3`.
define cube_suc(k: Nat) -> Nat {
    (k.suc).pow(Nat.3)
}

/// The (k+1)-th natural number, `k+1`.
define nat_suc(k: Nat) -> Nat {
    k.suc
}

/// The square of a sum expands: `(A + b)^2 = A^2 + 2 A b + b^2`.
theorem nicomachus_square_add(a: Nat, b: Nat) {
    (a + b) * (a + b) = a * a + (a + a) * b + b * b
} by {
    (a + b) * (a + b) = (a + b) * a + (a + b) * b
    (a + b) * a = a * a + b * a
    (a + b) * b = a * b + b * b
    (a + b) * (a + b) = a * a + b * a + (a * b + b * b)
    b * a = a * b
    a * a + b * a + (a * b + b * b) = a * a + (a * b + a * b) + b * b
    a * a + (a * b + a * b) + b * b = a * a + (a + a) * b + b * b
    (a + b) * (a + b) = a * a + (a + a) * b + b * b
}

/// The cube of a successor: `(x+1)^3 = x (x+1)^2 + (x+1)^2`.
theorem nicomachus_cube_expand(x: Nat) {
    (x.suc).pow(Nat.3) = (x * x + x) * x.suc + x.suc * x.suc
} by {
    (x.suc).pow(Nat.3) = x.suc * x.suc * x.suc
    x.suc * x.suc * x.suc = (x * x + x) * x.suc + x.suc * x.suc
    (x.suc).pow(Nat.3) = (x * x + x) * x.suc + x.suc * x.suc
}

/// Twice the sum of the first `n` naturals is `n^2 + n`.
theorem nicomachus_double_sum(n: Nat) {
    partial[Nat](nat_suc, n) + partial[Nat](nat_suc, n) = n * n + n
} by {
    define p(x: Nat) -> Bool {
        partial[Nat](nat_suc, x) + partial[Nat](nat_suc, x) = x * x + x
    }

    partial_zero[Nat](nat_suc)
    partial[Nat](nat_suc, Nat.0) = Nat.0
    Nat.0 * Nat.0 + Nat.0 = Nat.0
    p(Nat.0)

    forall(x: Nat) {
        if p(x) {
            partial[Nat](nat_suc, x) + partial[Nat](nat_suc, x) = x * x + x
            partial_split_last[Nat](nat_suc, x)
            partial[Nat](nat_suc, x.suc) = partial[Nat](nat_suc, x) + nat_suc(x)
            partial[Nat](nat_suc, x.suc) = partial[Nat](nat_suc, x) + x.suc
            partial[Nat](nat_suc, x.suc) + partial[Nat](nat_suc, x.suc) =
                (partial[Nat](nat_suc, x) + x.suc) + (partial[Nat](nat_suc, x) + x.suc)
            (partial[Nat](nat_suc, x) + x.suc) + (partial[Nat](nat_suc, x) + x.suc) =
                (partial[Nat](nat_suc, x) + partial[Nat](nat_suc, x)) + (x.suc + x.suc)
            (partial[Nat](nat_suc, x) + partial[Nat](nat_suc, x)) + (x.suc + x.suc) =
                (x * x + x) + (x.suc + x.suc)
            partial[Nat](nat_suc, x.suc) + partial[Nat](nat_suc, x.suc) =
                (x * x + x) + (x.suc + x.suc)
            (x * x + x) + (x.suc + x.suc) = x.suc * x.suc + x.suc
            partial[Nat](nat_suc, x.suc) + partial[Nat](nat_suc, x.suc) =
                x.suc * x.suc + x.suc
            p(x.suc)
        }
    }
    p(n)
}

/// Nicomachus's theorem: the sum of the first `n` cubes is the square of
/// the sum of the first `n` naturals.
theorem theorems1000_nicomachus(n: Nat) {
    partial[Nat](cube_suc, n) = partial[Nat](nat_suc, n) * partial[Nat](nat_suc, n)
} by {
    define p(x: Nat) -> Bool {
        partial[Nat](cube_suc, x) = partial[Nat](nat_suc, x) * partial[Nat](nat_suc, x)
    }

    partial_zero[Nat](cube_suc)
    partial[Nat](cube_suc, Nat.0) = Nat.0
    partial_zero[Nat](nat_suc)
    partial[Nat](nat_suc, Nat.0) = Nat.0
    Nat.0 * Nat.0 = Nat.0
    p(Nat.0)

    forall(x: Nat) {
        if p(x) {
            partial[Nat](cube_suc, x) = partial[Nat](nat_suc, x) * partial[Nat](nat_suc, x)
            partial_split_last[Nat](cube_suc, x)
            partial[Nat](cube_suc, x.suc) = partial[Nat](cube_suc, x) + cube_suc(x)
            partial[Nat](cube_suc, x.suc) =
                partial[Nat](nat_suc, x) * partial[Nat](nat_suc, x) + cube_suc(x)
            partial_split_last[Nat](nat_suc, x)
            partial[Nat](nat_suc, x.suc) = partial[Nat](nat_suc, x) + nat_suc(x)
            partial[Nat](nat_suc, x.suc) = partial[Nat](nat_suc, x) + x.suc
            nicomachus_square_add(partial[Nat](nat_suc, x), x.suc)
            partial[Nat](nat_suc, x.suc) * partial[Nat](nat_suc, x.suc) =
                partial[Nat](nat_suc, x) * partial[Nat](nat_suc, x) +
                (partial[Nat](nat_suc, x) + partial[Nat](nat_suc, x)) * x.suc +
                x.suc * x.suc
            nicomachus_double_sum(x)
            partial[Nat](nat_suc, x) + partial[Nat](nat_suc, x) = x * x + x
            partial[Nat](nat_suc, x.suc) * partial[Nat](nat_suc, x.suc) =
                partial[Nat](nat_suc, x) * partial[Nat](nat_suc, x) +
                (x * x + x) * x.suc + x.suc * x.suc
            nicomachus_cube_expand(x)
            (x.suc).pow(Nat.3) = (x * x + x) * x.suc + x.suc * x.suc
            cube_suc(x) = (x.suc).pow(Nat.3)
            cube_suc(x) = (x * x + x) * x.suc + x.suc * x.suc
            partial[Nat](cube_suc, x.suc) =
                partial[Nat](nat_suc, x) * partial[Nat](nat_suc, x) +
                (x * x + x) * x.suc + x.suc * x.suc
            partial[Nat](cube_suc, x.suc) =
                partial[Nat](nat_suc, x.suc) * partial[Nat](nat_suc, x.suc)
            p(x.suc)
        }
    }
    p(n)
}
