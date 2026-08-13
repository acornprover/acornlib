from nat import Nat, add_sub, add_comm, add_assoc, mul_comm, lt_imp_lte_suc, lte_mul_both,
    lte_add_left, mul_two_left, mul_one_right, distrib_left, sub_pos, lte_trans, lte_ref,
    sub_self, sub_lt, mul_to_zero, add_imp_sub, add_imp_sub_left, lte_imp_not_lt,
    lt_or_lte, divides_lte, mul_one_left, lte_antisymm, lt_trans, sum_lte, lte_mul,
    lt_add_left, lt_cancel_mul, lt_mul_both,
    div_mod_decomp, mod_lt, div_imp_mod, divides_self, divides_mul, mod_lte,
    mul_assoc, distrib_right, sub_comm
from number_theory.interface import congr_mod_refl, congr_mod_trans, congr_mod_mul,
    mod_congr_mod_self, coprime_comm, mod_inv, gcd_mult_right, gcd_divides

numerals Nat

// The Frobenius coin problem (named after Georg Frobenius; the
// two-coin case was known to James Joseph Sylvester in 1884): for coprime
// coin denominations `a`, `b` (both greater than one), the largest amount
// that cannot be paid with nonnegative numbers of `a`-coins and `b`-coins is
//     ab - a - b.
// Every amount larger than `ab - a - b` IS payable, and `ab - a - b` itself
// is not.  The theorem is also called the "Chicken McNugget theorem".
//
// Proof (two parts).
//
// Non-representability of `ab - a - b`: suppose `a x + b y = ab - a - b`.
// Adding `a + b` to both sides gives `a (x + 1) + b (y + 1) = ab`.  We first
// note `x + 1 <= b` (otherwise `a (x + 1) >= a (b + 1) = ab + a > ab`), so
// `b (y + 1) = ab - a (x + 1) = a (b - (x + 1))`, hence `a | b (y + 1)` and
// by Euclid's lemma `a | (y + 1)`, so `y + 1 >= a`.  Then
// `b (y + 1) >= b a = ab`, while `b (y + 1) = ab - a (x + 1) <= ab`, so
// `b (y + 1) = ab` and `a (x + 1) = 0`, contradicting `x + 1 >= 1`.
//
// Representability of every `n > ab - a - b`: let `v` be the modular
// inverse of `b` modulo `a` (it exists because `gcd(a, b) = 1`).  Take
// `y = (n v) mod a < a`; then `b y ≡ n (mod a)`, so `b y` and `n` have the
// same remainder `r` modulo `a`: `b y = q1 a + r` and `n = q2 a + r`.  We
// claim `q1 <= q2` (i.e. `b y <= n`): otherwise `q2 < q1` would give
// `n = q2 a + r < q1 a + r = b y`.  Then `n - b y = (q2 - q1) a` is a
// multiple of `a`, so `n = a (q2 - q1) + b y` is representable.  If instead
// `n < b y`, then `q2 < q1`, so `b y = n + (q1 - q2) a >= n + a`, and since
// `y < a` gives `b y <= b (a - 1) = ab - b`, we get `n + a <= ab - b`,
// hence `n <= ab - a - b`, contradicting `n > ab - a - b`.

/// `b <= a * b - a` for `1 < a` and `1 < b`: subtracting one `a` from the
/// product leaves at least `b`.
theorem frob_bound_b(a: Nat, b: Nat) {
    Nat.1 < a and Nat.1 < b implies b <= a * b - a
} by {
    if Nat.1 < a and Nat.1 < b {
        Nat.1 < a
        Nat.1 < b
        lt_imp_lte_suc(Nat.1, a)
        Nat.2 <= a
        // 1 <= b - 1
        sub_pos(b, Nat.1)
        b - Nat.1 > Nat.0
        Nat.0 < b - Nat.1
        lt_imp_lte_suc(Nat.0, b - Nat.1)
        Nat.1 <= b - Nat.1
        // (b-1) + 1 <= (b-1) + (b-1)
        lte_add_left(b - Nat.1, Nat.1, b - Nat.1)
        (b - Nat.1) + Nat.1 <= (b - Nat.1) + (b - Nat.1)
        // b = (b-1) + 1
        add_sub(b, Nat.1)
        b - Nat.1 + Nat.1 = b
        b <= (b - Nat.1) + (b - Nat.1)
        // (b-1) + (b-1) = 2(b-1) = (b-1) * 2
        mul_two_left(b - Nat.1)
        Nat.2 * (b - Nat.1) = (b - Nat.1) + (b - Nat.1)
        mul_comm(Nat.2, b - Nat.1)
        Nat.2 * (b - Nat.1) = (b - Nat.1) * Nat.2
        (b - Nat.1) + (b - Nat.1) = (b - Nat.1) * Nat.2
        b <= (b - Nat.1) * Nat.2
        // (b-1) * 2 <= (b-1) * a
        lte_mul_both(b - Nat.1, Nat.2, a)
        (b - Nat.1) * Nat.2 <= (b - Nat.1) * a
        lte_trans(b, (b - Nat.1) * Nat.2, (b - Nat.1) * a)
        b <= (b - Nat.1) * a
        // a(b-1) = ab - a, and (b-1) a = a (b-1)
        mul_comm(a, b - Nat.1)
        a * (b - Nat.1) = (b - Nat.1) * a
        distrib_left(a, b - Nat.1, Nat.1)
        a * (b - Nat.1) = a * b - a * Nat.1
        mul_one_right(a)
        a * Nat.1 = a
        a * (b - Nat.1) = a * b - a
        (b - Nat.1) * a = a * b - a
        lte_trans(b, (b - Nat.1) * a, a * b - a)
        b <= a * b - a
    }
}

/// `a <= a * b` for `1 < b`: the product is at least one `a`.
theorem frob_bound_a(a: Nat, b: Nat) {
    Nat.1 < b implies a <= a * b
} by {
    if Nat.1 < b {
        lt_trans(Nat.0, Nat.1, b)
        Nat.0 < b
        b != Nat.0
        lte_mul(a, b)
        a <= a * b
    }
}

/// `(a * b - a - b) + a + b = a * b`: adding the subtracted amount back.
theorem frob_sum_ab(a: Nat, b: Nat) {
    Nat.1 < a and Nat.1 < b implies (a * b - a - b) + a + b = a * b
} by {
    if Nat.1 < a and Nat.1 < b {
        frob_bound_b(a, b)
        b <= a * b - a
        add_sub(a * b - a, b)
        (a * b - a) - b + b = a * b - a
        frob_bound_a(a, b)
        a <= a * b
        add_sub(a * b, a)
        (a * b - a) + a = a * b
        add_assoc(a * b - a - b, b, a)
        (a * b - a - b) + b + a = (a * b - a - b) + (b + a)
        add_comm(b, a)
        b + a = a + b
        (a * b - a - b) + b + a = (a * b - a - b) + (a + b)
        (a * b - a - b) + b + a = (a * b - a) + a
        (a * b - a - b) + b + a = a * b
        (a * b - a - b) + (a + b) = a * b
    }
}

/// `a * b <= b * (y + 1)` when `y + 1 >= a`.
theorem frob_mul_ge(a: Nat, b: Nat, y: Nat) {
    a <= y + Nat.1 implies a * b <= b * (y + Nat.1)
} by {
    if a <= y + Nat.1 {
        lte_mul_both(b, a, y + Nat.1)
        b * a <= b * (y + Nat.1)
        mul_comm(b, a)
        b * a = a * b
        a * b <= b * (y + Nat.1)
    }
}

/// A larger number minus a smaller one is zero.
theorem frob_le_sub_zero(m: Nat, n: Nat) {
    m <= n implies m - n = Nat.0
} by {
    if m <= n {
        if m < n {
            sub_lt(m, n)
            m - n = Nat.0
        }
        if not m < n {
            lt_or_lte(m, n)
            n <= m
            lte_antisymm(m, n)
            m = n
            sub_self(m)
            m - m = Nat.0
            m - n = Nat.0
        }
        m - n = Nat.0
    }
}

/// Subtracting anything from a number leaves at most the number.
theorem frob_sub_lte_self(m: Nat, k: Nat) {
    m - k <= m
} by {
    if k <= m {
        add_sub(m, k)
        m - k + k = m
        sum_lte(m - k, Nat.0, m - k, k)
        (m - k) + Nat.0 <= (m - k) + k
        (m - k) + Nat.0 = m - k
        m - k <= m
    }
    if not k <= m {
        m < k
        sub_lt(m, k)
        m - k = Nat.0
        Nat.0 <= m
        m - k <= m
    }
}

/// Adding the same amount to both sides preserves strict inequality on the
/// left: `k + a < n + a` when `k < n`.
theorem frob_lt_add_left(a: Nat, k: Nat, n: Nat) {
    k < n implies k + a < n + a
} by {
    if k < n {
        lt_add_left(a, k, n)
        a + k < a + n
        add_comm(a, k)
        a + k = k + a
        add_comm(a, n)
        a + n = n + a
        k + a < n + a
    }
}

/// Cancelling a common addend on the right of an inequality:
/// `n + a <= k + a` implies `n <= k`.
theorem frob_lte_add_cancel(n: Nat, k: Nat, a: Nat) {
    n + a <= k + a implies n <= k
} by {
    if n + a <= k + a {
        if n <= k {
        }
        if not n <= k {
            k < n
            frob_lt_add_left(a, k, n)
            k + a < n + a
            add_comm(k, a)
            k + a = a + k
            add_comm(n, a)
            n + a = a + n
            a + k < a + n
            // a + k < a + n and a + n <= a + k: contradiction
            lte_imp_not_lt(a + k, a + n)
            false
        }
        n <= k
    }
}

/// From `n + a <= m` follows `n <= m - a`.
theorem frob_add_lte_imp_sub(n: Nat, a: Nat, m: Nat) {
    n + a <= m implies n <= m - a
} by {
    if n + a <= m {
        // a <= m from a <= n + a <= m
        a <= n + a
        lte_trans(a, n + a, m)
        a <= m
        add_sub(m, a)
        m - a + a = m
        n + a <= (m - a) + a
        frob_lte_add_cancel(n, m - a, a)
        n <= m - a
    }
}

/// Euclid's lemma: if `a` is coprime to `b` and divides `b * c`, then it
/// divides `c`.
theorem frob_euclid(a: Nat, b: Nat, c: Nat) {
    a.coprime(b) and a.divides(b * c) implies a.divides(c)
} by {
    if a.coprime(b) and a.divides(b * c) {
        a.coprime(b) = (a.gcd(b) = Nat.1)
        a.gcd(b) = Nat.1
        gcd_mult_right(a, b, c)
        a.gcd(b) * c = (a * c).gcd(b * c)
        Nat.1 * c = (a * c).gcd(b * c)
        mul_one_left(c)
        Nat.1 * c = c
        c = (a * c).gcd(b * c)
        divides_self(a)
        a.divides(a)
        divides_mul(a, c, a)
        a.divides(a * c)
        a.divides(b * c)
        gcd_divides(a, a * c, b * c)
        a.divides((a * c).gcd(b * c))
        a.divides(c)
    }
}

/// The two-coin Frobenius number: with coprime denominations `a`, `b > 1`,
/// the amount `ab - a - b` cannot be represented as `a x + b y`.
theorem frob_not_representable(a: Nat, b: Nat) {
    Nat.1 < a and Nat.1 < b and a.coprime(b) implies
        not exists(x: Nat, y: Nat) { a * x + b * y = a * b - a - b }
} by {
    if Nat.1 < a and Nat.1 < b and a.coprime(b) {
        Nat.1 < a
        Nat.1 < b
        a.coprime(b)
        if exists(x: Nat, y: Nat) { a * x + b * y = a * b - a - b } {
            let (x: Nat, y: Nat) satisfy { a * x + b * y = a * b - a - b }
            a * x + b * y = a * b - a - b
            // a(x+1) + b(y+1) = (ax + by) + (a + b) = ab
            distrib_left(a, x, Nat.1)
            a * (x + Nat.1) = a * x + a * Nat.1
            mul_one_right(a)
            a * Nat.1 = a
            a * (x + Nat.1) = a * x + a
            distrib_left(b, y, Nat.1)
            b * (y + Nat.1) = b * y + b * Nat.1
            mul_one_right(b)
            b * Nat.1 = b
            b * (y + Nat.1) = b * y + b
            add_assoc(a * x, a, b * y + b)
            a * x + a + (b * y + b) = a * x + (a + (b * y + b))
            add_comm(a, b * y)
            a + b * y = b * y + a
            add_assoc(a * x, b * y, a)
            a * x + (b * y + a) = (a * x + b * y) + a
            a * x + a + (b * y + b) = (a * x + b * y) + (a + b)
            a * (x + Nat.1) + b * (y + Nat.1) =
                (a * x + b * y) + (a + b)
            frob_sum_ab(a, b)
            (a * b - a - b) + a + b = a * b
            a * (x + Nat.1) + b * (y + Nat.1) = a * b
            // x + 1 <= b
            a * (x + Nat.1) = a * b - b * (y + Nat.1)
            add_imp_sub(a * (x + Nat.1), b * (y + Nat.1), a * b)
            a * b - b * (y + Nat.1) = a * (x + Nat.1)
            frob_sub_lte_self(a * b, b * (y + Nat.1))
            a * b - b * (y + Nat.1) <= a * b
            a * (x + Nat.1) <= a * b
            if b < x + Nat.1 {
                lt_imp_lte_suc(b, x + Nat.1)
                b + Nat.1 <= x + Nat.1
                lte_mul_both(a, b + Nat.1, x + Nat.1)
                a * (b + Nat.1) <= a * (x + Nat.1)
                lte_trans(a * (b + Nat.1), a * (x + Nat.1), a * b)
                a * (b + Nat.1) <= a * b
                distrib_left(a, b, Nat.1)
                a * (b + Nat.1) = a * b + a * Nat.1
                mul_one_right(a)
                a * Nat.1 = a
                a * (b + Nat.1) = a * b + a
                lt_trans(Nat.0, Nat.1, a)
                Nat.0 < a
                lt_add_left(a * b, Nat.0, a)
                a * b + Nat.0 < a * b + a
                a * b + Nat.0 = a * b
                a * b < a * b + a
                a * b < a * (b + Nat.1)
                lte_imp_not_lt(a * (b + Nat.1), a * b)
                false
            }
            not b < x + Nat.1
            lt_or_lte(b, x + Nat.1)
            x + Nat.1 <= b
            // b(y+1) = ab - a(x+1) = a(b - (x+1))
            add_sub(b, x + Nat.1)
            b - (x + Nat.1) + (x + Nat.1) = b
            distrib_left(a, b - (x + Nat.1), x + Nat.1)
            a * (b - (x + Nat.1) + (x + Nat.1)) =
                a * (b - (x + Nat.1)) + a * (x + Nat.1)
            a * b = a * (b - (x + Nat.1)) + a * (x + Nat.1)
            add_imp_sub(a * (b - (x + Nat.1)), a * (x + Nat.1), a * b)
            a * b - a * (x + Nat.1) = a * (b - (x + Nat.1))
            add_imp_sub_left(a * (x + Nat.1), b * (y + Nat.1), a * b)
            a * b - a * (x + Nat.1) = b * (y + Nat.1)
            b * (y + Nat.1) = a * (b - (x + Nat.1))
            // a | b(y+1)
            a.divides(b * (y + Nat.1)) =
                exists(c: Nat) { a * c = b * (y + Nat.1) }
            exists(c: Nat) { a * c = b * (y + Nat.1) }
            a.divides(b * (y + Nat.1))
            // Euclid: a | (y+1)
            frob_euclid(a, b, y + Nat.1)
            a.divides(y + Nat.1)
            divides_lte(a, y + Nat.1)
            y + Nat.1 = Nat.0 or a <= y + Nat.1
            y + Nat.1 != Nat.0
            a <= y + Nat.1
            // ab <= b(y+1) and b(y+1) <= ab, so b(y+1) = ab
            frob_mul_ge(a, b, y)
            a * b <= b * (y + Nat.1)
            b * (y + Nat.1) = a * b - a * (x + Nat.1)
            frob_sub_lte_self(a * b, a * (x + Nat.1))
            a * b - a * (x + Nat.1) <= a * b
            b * (y + Nat.1) <= a * b
            lte_antisymm(a * b, b * (y + Nat.1))
            b * (y + Nat.1) = a * b
            // a(x+1) = 0
            add_imp_sub(a * (x + Nat.1), b * (y + Nat.1), a * b)
            a * b - b * (y + Nat.1) = a * (x + Nat.1)
            sub_self(a * b)
            a * b - a * b = Nat.0
            b * (y + Nat.1) = a * b
            a * b - b * (y + Nat.1) = Nat.0
            a * (x + Nat.1) = Nat.0
            mul_to_zero(a, x + Nat.1)
            a = Nat.0 or x + Nat.1 = Nat.0
            x + Nat.1 != Nat.0
            false
        }
    }
}

/// Every `n > ab - a - b` is representable as `a x + b y`.
theorem frob_representable(a: Nat, b: Nat) {
    Nat.1 < a and Nat.1 < b and a.coprime(b) implies
        forall(n: Nat) {
            a * b - a - b < n implies
                exists(x: Nat, y: Nat) { a * x + b * y = n }
        }
} by {
    if Nat.1 < a and Nat.1 < b and a.coprime(b) {
        Nat.1 < a
        Nat.1 < b
        a.coprime(b)
        a != Nat.0
        b != Nat.0
        forall(n: Nat) {
            if a * b - a - b < n {
                let v: Nat = mod_inv(b, a)
                coprime_comm(a, b)
                b.coprime(a)
                (b * mod_inv(b, a)).congr_mod(Nat.1, a)
                (b * v).congr_mod(Nat.1, a)
                // y = (n * v) mod a
                let y: Nat = (n * v).mod(a)
                mod_lt(n * v, a)
                y < a
                // b*y ≡ n (mod a)
                mod_congr_mod_self(n * v, a)
                (n * v).mod(a).congr_mod(n * v, a)
                y.congr_mod(n * v, a)
                congr_mod_refl(b, a)
                b.congr_mod(b, a)
                congr_mod_mul(b, y, b, n * v, a)
                (b * y).congr_mod(b * (n * v), a)
                // b(nv) = n(bv)
                mul_assoc(n, v, b)
                (n * v) * b = n * (v * b)
                mul_comm(v, b)
                v * b = b * v
                n * (v * b) = n * (b * v)
                (n * v) * b = n * (b * v)
                mul_comm(n * v, b)
                (n * v) * b = b * (n * v)
                b * (n * v) = n * (b * v)
                congr_mod_refl(n, a)
                n.congr_mod(n, a)
                congr_mod_mul(n, b * v, n, Nat.1, a)
                (n * (b * v)).congr_mod(n * Nat.1, a)
                mul_one_right(n)
                n * Nat.1 = n
                (n * (b * v)).congr_mod(n, a)
                congr_mod_trans(b * y, b * (n * v), n, a)
                (b * y).congr_mod(n, a)
                // b*y and n have the same remainder r mod a
                // b*y = q1*a + r, n = q2*a + r
                div_mod_decomp(b * y, a)
                (b * y).div(a) * a + (b * y).mod(a) = b * y
                div_mod_decomp(n, a)
                n.div(a) * a + n.mod(a) = n
                (b * y).congr_mod(n, a)
                (b * y).mod(a) = n.mod(a)
                (b * y).div(a) * a + n.mod(a) = b * y
                n.div(a) * a + n.mod(a) = n
                // case 1: q1 <= q2 (b*y <= n)
                if (b * y).div(a) <= n.div(a) {
                    // n = (q2 - q1)*a + by, so n - by = (q2 - q1)*a
                    add_sub(n.div(a), (b * y).div(a))
                    n.div(a) - (b * y).div(a) + (b * y).div(a) = n.div(a)
                    distrib_right(n.div(a) - (b * y).div(a), (b * y).div(a), a)
                    (n.div(a) - (b * y).div(a) + (b * y).div(a)) * a =
                        (n.div(a) - (b * y).div(a)) * a + (b * y).div(a) * a
                    n.div(a) * a =
                        (n.div(a) - (b * y).div(a)) * a + (b * y).div(a) * a
                    n.div(a) * a + n.mod(a) = n
                    (n.div(a) - (b * y).div(a)) * a + (b * y).div(a) * a +
                        n.mod(a) = n
                    (n.div(a) - (b * y).div(a)) * a +
                        ((b * y).div(a) * a + n.mod(a)) = n
                    (b * y).div(a) * a + n.mod(a) = b * y
                    (n.div(a) - (b * y).div(a)) * a + b * y = n
                    add_imp_sub((n.div(a) - (b * y).div(a)) * a, b * y, n)
                    n - b * y = (n.div(a) - (b * y).div(a)) * a
                    // exists x: a*x = n - by
                    let qdiff: Nat = n.div(a) - (b * y).div(a)
                    a * qdiff = n - b * y
                    exists(c: Nat) { a * c = n - b * y }
                    a.divides(n - b * y) =
                        exists(c: Nat) { a * c = n - b * y }
                    a.divides(n - b * y)
                    let (w: Nat) satisfy { a * w = n - b * y }
                    a * w = n - b * y
                    add_sub(n, b * y)
                    n - b * y + b * y = n
                    a * w + b * y = n
                    exists(px: Nat, py: Nat) { a * px + b * py = n }
                }
                // case 2: q1 > q2 -- contradiction with n > ab - a - b
                if not (b * y).div(a) <= n.div(a) {
                    lt_or_lte(n.div(a), (b * y).div(a))
                    n.div(a) < (b * y).div(a)
                    // q1 = q2 + d with d >= 1
                    add_sub((b * y).div(a), n.div(a))
                    (b * y).div(a) - n.div(a) + n.div(a) = (b * y).div(a)
                    let d: Nat = (b * y).div(a) - n.div(a)
                    d + n.div(a) = (b * y).div(a)
                    // d >= 1
                    if d = Nat.0 {
                        Nat.0 + n.div(a) = (b * y).div(a)
                        n.div(a) = (b * y).div(a)
                        false
                    }
                    d != Nat.0
                    lt_imp_lte_suc(Nat.0, d)
                    Nat.1 <= d
                    // by = q1*a + r = (d + q2)*a + r = d*a + (q2*a + r) = d*a + n
                    distrib_right(d, n.div(a), a)
                    (d + n.div(a)) * a = d * a + n.div(a) * a
                    (b * y).div(a) * a = d * a + n.div(a) * a
                    (b * y).div(a) * a + n.mod(a) = d * a + n.div(a) * a + n.mod(a)
                    // by = (q1)*a + r and n = q2*a + r
                    (b * y).div(a) * a + n.mod(a) = b * y
                    n.div(a) * a + n.mod(a) = n
                    d * a + n.div(a) * a + n.mod(a) = b * y
                    d * a + (n.div(a) * a + n.mod(a)) = b * y
                    d * a + n = b * y
                    n + d * a = b * y
                    // d*a >= a, so n + a <= n + d*a = by
                    lte_mul(a, d)
                    a <= a * d
                    mul_comm(a, d)
                    a * d = d * a
                    a <= d * a
                    lte_add_left(n, a, d * a)
                    n + a <= n + d * a
                    n + a <= b * y
                    // y <= a - 1, so b*y <= b(a-1) = ab - b
                    lt_imp_lte_suc(y, a)
                    y + Nat.1 <= a
                    frob_add_lte_imp_sub(y, Nat.1, a)
                    y <= a - Nat.1
                    lte_mul_both(b, y, a - Nat.1)
                    b * y <= b * (a - Nat.1)
                    distrib_left(b, a - Nat.1, Nat.1)
                    b * ((a - Nat.1) + Nat.1) = b * (a - Nat.1) + b * Nat.1
                    add_sub(a, Nat.1)
                    a - Nat.1 + Nat.1 = a
                    b * a = b * (a - Nat.1) + b * Nat.1
                    mul_one_right(b)
                    b * Nat.1 = b
                    b * a = b * (a - Nat.1) + b
                    add_imp_sub(b * (a - Nat.1), b, b * a)
                    b * a - b = b * (a - Nat.1)
                    b * (a - Nat.1) = b * a - b
                    mul_comm(b, a)
                    b * a = a * b
                    b * (a - Nat.1) = a * b - b
                    lte_trans(n + a, b * y, a * b - b)
                    n + a <= a * b - b
                    frob_add_lte_imp_sub(n, a, a * b - b)
                    n <= a * b - b - a
                    // (ab - b) - a = ab - a - b
                    frob_bound_b(a, b)
                    b <= a * b - a
                    add_sub(a * b, a)
                    a * b - a + a = a * b
                    sum_lte(b, a, a * b - a, a)
                    b + a <= (a * b - a) + a
                    add_comm(b, a)
                    b + a = a + b
                    a + b <= a * b
                    sub_comm(a * b, b, a)
                    (a * b - b) - a = (a * b - a) - b
                    n <= a * b - a - b
                    // n > ab - a - b: contradiction
                    lte_imp_not_lt(a * b - a - b, n)
                    false
                }
                exists(px: Nat, py: Nat) { a * px + b * py = n }
            }
        }
    }
}

/// The Frobenius coin problem for two denominations.
theorem theorems1000_frobenius_coin(a: Nat, b: Nat) {
    Nat.1 < a and Nat.1 < b and a.coprime(b) implies
        (not exists(x: Nat, y: Nat) { a * x + b * y = a * b - a - b }) and
        forall(n: Nat) {
            a * b - a - b < n implies
                exists(x: Nat, y: Nat) { a * x + b * y = n }
        }
} by {
    if Nat.1 < a and Nat.1 < b and a.coprime(b) {
        frob_not_representable(a, b)
        not exists(x: Nat, y: Nat) { a * x + b * y = a * b - a - b }
        frob_representable(a, b)
        (forall(n: Nat) {
            a * b - a - b < n implies
                exists(x: Nat, y: Nat) { a * x + b * y = n }
        })
        (not exists(x: Nat, y: Nat) { a * x + b * y = a * b - a - b }) and
        (forall(n: Nat) {
            a * b - a - b < n implies
                exists(x: Nat, y: Nat) { a * x + b * y = n }
        })
    }
}
