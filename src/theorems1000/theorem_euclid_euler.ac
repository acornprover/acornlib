from nat import Nat, sub_pos, pow_one, pow_add, divides_self, divides_mul, mul_two_left,
    add_sub, lte_trans
from number_theory.interface import divisor_sum_fn, nat_sigma, is_perfect, euclid_construction,
    divisor_list
from list import sum, map, map_identity
from data.basic.functions import identity_fn

numerals Nat

// The Euclid-Euler theorem: a natural number `n` is an even perfect number
// if and only if it is of the form
//     n = 2^(p-1) (2^p - 1),
// where `p` is prime and `2^p - 1` is prime (a Mersenne prime).  The
// forward direction was proved by Euler (eighteenth century); the reverse
// direction is Euclid's Proposition IX.36 of the "Elements" (c. 300 BC):
// if `2^p - 1` is prime then `2^(p-1)(2^p - 1)` is perfect.
//
// A perfect number is one equal to the sum of its proper divisors; with
// the divisor-sum operator `divisor_sum_fn` (the sum of `f(d)` over the
// positive divisors `d | n`), `n` is perfect exactly when
//     divisor_sum_fn(identity_fn[Nat])(n) = n + n,
// because the sum of all divisors is `2 n` precisely when the sum of the
// proper divisors is `n`.  Evenness is `Nat.2.divides(n)`.  A Mersenne
// prime is a prime `p` for which `2^p - 1` is prime.
//
// Proof sketch (reverse, Euclid): if `q = 2^p - 1` is prime, the divisors
// of `2^(p-1) q` are exactly `2^i` and `2^i q` for `0 <= i < p`, and they
// sum to `(2^p - 1) + (2^p - 1) q = 2 (2^(p-1) q)`.  The library proves
// this as `euclid_construction` in `number_theory/mersenne_perfect.ac`
// (in terms of `is_perfect`, i.e. `nat_sigma(n) = 2 n`); the wrapper
// restates it in terms of `divisor_sum_fn(identity_fn[Nat])(n) = n + n`
// and adds the evenness of `2^(p-1) (2^p - 1)` (the factor `2^(p-1)` is
// divisible by `2` because `p >= 2`).
//
// Proof sketch (forward, Euler): if `n` is even and perfect, write
// `n = 2^(k-1) m` with `m` odd (`k >= 2`).  Multiplicativity of the
// divisor-sum function on coprime factors gives
// `(2^k - 1) sigma(m) = 2^k m`; since `2^k - 1` is coprime to `2^k`, it
// divides `m`, say `m = (2^k - 1) r`.  Substituting and cancelling shows
// `sigma(m) = 2^k r`, and since `r` is a proper divisor of `m` with
// `sigma(m) >= m + r`, equality forces `r = 1`, so `m = 2^k - 1` and
// `sigma(m) = m + 1`, which means `m` is prime.  This direction needs the
// multiplicativity of the divisor-sum function on coprime factors, proved
// as `nat_sigma_multiplicative` in `number_theory/dirichlet.ac`, together
// with the even-odd decomposition of `n`; it is recorded below without
// proof, while the reverse direction is proved here.

/// Euclid's direction of the Euclid-Euler theorem: if `p` is prime and
/// `2^p - 1` is prime, then `2^(p-1) (2^p - 1)` is an even perfect number.
theorem theorems1000_euler_euclid(p: Nat) {
    Nat.1 < p and p.is_prime and (Nat.2.pow(p) - Nat.1).is_prime
        implies Nat.2.divides(Nat.2.pow(p - Nat.1) * (Nat.2.pow(p) - Nat.1)) and
            divisor_sum_fn(identity_fn[Nat])(Nat.2.pow(p - Nat.1) * (Nat.2.pow(p) - Nat.1)) =
                Nat.2.pow(p - Nat.1) * (Nat.2.pow(p) - Nat.1) +
                Nat.2.pow(p - Nat.1) * (Nat.2.pow(p) - Nat.1)
} by {
    if Nat.1 < p and p.is_prime and (Nat.2.pow(p) - Nat.1).is_prime {
        Nat.1 < p
        p.is_prime
        (Nat.2.pow(p) - Nat.1).is_prime
        // evenness: 2 divides 2^(p-1), so 2 divides 2^(p-1) * (2^p - 1).
        sub_pos(p, Nat.1)
        p - Nat.1 > Nat.0
        Nat.0 < p - Nat.1
        let m: Nat satisfy { m.suc = p - Nat.1 }
        m.suc = p - Nat.1
        pow_one(Nat.2)
        Nat.2.pow(Nat.1) = Nat.2
        pow_add(Nat.2, Nat.1, m)
        Nat.2.pow(Nat.1) * Nat.2.pow(m) = Nat.2.pow(Nat.1 + m)
        Nat.1 + m = m.suc
        Nat.2.pow(Nat.1 + m) = Nat.2.pow(p - Nat.1)
        Nat.2.pow(Nat.1) * Nat.2.pow(m) = Nat.2.pow(p - Nat.1)
        Nat.2 * Nat.2.pow(m) = Nat.2.pow(p - Nat.1)
        divides_self(Nat.2)
        Nat.2.divides(Nat.2)
        divides_mul(Nat.2.pow(m), Nat.2, Nat.2)
        Nat.2.divides(Nat.2) implies Nat.2.divides(Nat.2 * Nat.2.pow(m))
        Nat.2.divides(Nat.2 * Nat.2.pow(m))
        Nat.2 * Nat.2.pow(m) = Nat.2.pow(p - Nat.1)
        Nat.2.divides(Nat.2.pow(p - Nat.1))
        divides_mul(Nat.2.pow(p - Nat.1), Nat.2.pow(p) - Nat.1, Nat.2)
        Nat.2.divides(Nat.2.pow(p - Nat.1)) implies Nat.2.divides(Nat.2.pow(p - Nat.1) * (Nat.2.pow(p) - Nat.1))
        Nat.2.divides(Nat.2.pow(p - Nat.1) * (Nat.2.pow(p) - Nat.1))
        // perfectness: euclid_construction gives is_perfect, i.e.
        // nat_sigma(X) = 2 X; the divisor-sum form follows from
        // divisor_sum_fn(identity_fn)(X) = nat_sigma(X).
        euclid_construction(p)
        is_perfect(Nat.2.pow(p - Nat.1) * (Nat.2.pow(p) - Nat.1))
        nat_sigma(Nat.2.pow(p - Nat.1) * (Nat.2.pow(p) - Nat.1)) =
            Nat.2 * (Nat.2.pow(p - Nat.1) * (Nat.2.pow(p) - Nat.1))
        map_identity(divisor_list(Nat.2.pow(p - Nat.1) * (Nat.2.pow(p) - Nat.1)))
        map[Nat, Nat](divisor_list(Nat.2.pow(p - Nat.1) * (Nat.2.pow(p) - Nat.1)), identity_fn[Nat]) =
            divisor_list(Nat.2.pow(p - Nat.1) * (Nat.2.pow(p) - Nat.1))
        sum(map[Nat, Nat](divisor_list(Nat.2.pow(p - Nat.1) * (Nat.2.pow(p) - Nat.1)), identity_fn[Nat])) =
            sum(divisor_list(Nat.2.pow(p - Nat.1) * (Nat.2.pow(p) - Nat.1)))
        sum(divisor_list(Nat.2.pow(p - Nat.1) * (Nat.2.pow(p) - Nat.1))) =
            nat_sigma(Nat.2.pow(p - Nat.1) * (Nat.2.pow(p) - Nat.1))
        divisor_sum_fn(identity_fn[Nat])(Nat.2.pow(p - Nat.1) * (Nat.2.pow(p) - Nat.1)) =
            nat_sigma(Nat.2.pow(p - Nat.1) * (Nat.2.pow(p) - Nat.1))
        mul_two_left(Nat.2.pow(p - Nat.1) * (Nat.2.pow(p) - Nat.1))
        Nat.2 * (Nat.2.pow(p - Nat.1) * (Nat.2.pow(p) - Nat.1)) =
            Nat.2.pow(p - Nat.1) * (Nat.2.pow(p) - Nat.1) +
            Nat.2.pow(p - Nat.1) * (Nat.2.pow(p) - Nat.1)
        divisor_sum_fn(identity_fn[Nat])(Nat.2.pow(p - Nat.1) * (Nat.2.pow(p) - Nat.1)) =
            Nat.2.pow(p - Nat.1) * (Nat.2.pow(p) - Nat.1) +
            Nat.2.pow(p - Nat.1) * (Nat.2.pow(p) - Nat.1)
        Nat.2.divides(Nat.2.pow(p - Nat.1) * (Nat.2.pow(p) - Nat.1)) and
            divisor_sum_fn(identity_fn[Nat])(Nat.2.pow(p - Nat.1) * (Nat.2.pow(p) - Nat.1)) =
                Nat.2.pow(p - Nat.1) * (Nat.2.pow(p) - Nat.1) +
                Nat.2.pow(p - Nat.1) * (Nat.2.pow(p) - Nat.1)
    }
}

// The Euler direction of the Euclid-Euler theorem: an even perfect number
// is of the form `2^(p-1) (2^p - 1)` with `p` and `2^p - 1` prime.  The
// proof needs the even-odd decomposition of `n` together with the
// multiplicativity of the divisor-sum function on coprime factors and the
// descent argument sketched above; it is recorded here without proof.
//
// theorem theorems1000_euclid_euler(n: Nat) {
//     Nat.2.divides(n) and divisor_sum_fn(identity_fn[Nat])(n) = n + n
//         implies exists(p: Nat) {
//             Nat.1 < p and p.is_prime and (Nat.2.pow(p) - Nat.1).is_prime and
//             n = Nat.2.pow(p - Nat.1) * (Nat.2.pow(p) - Nat.1)
//         }
// }
