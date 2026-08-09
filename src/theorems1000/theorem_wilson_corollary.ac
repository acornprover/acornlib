from nat import Nat, add_imp_sub, mul_one_right, suc_sub_one, add_sub, sub_pos,
    gcd_divides_left, gcd_divides_right, divides_sub, lte_ref, lte_trans, factorial_step,
    lt_suc, lt_trans
from number_theory.interface import prime_imp_wilson_factorial_congr, cancel_coprime,
    coprime_comm, nat_divides_one_imp_one

numerals Nat

// A corollary of Wilson's theorem: for every prime `p`,
//     (p - 2)! ≡ 1  (mod p).
// Wilson's theorem itself (proved in `number_theory/wilson.ac`) says
//     (p - 1)! ≡ p - 1 ≡ -1  (mod p).
// Since (p - 1)! = (p - 1) * (p - 2)!, the congruence reads
//     (p - 1) * (p - 2)! ≡ (p - 1) * 1  (mod p),
// and cancelling the factor `p - 1` — which is coprime to `p`, because any
// common divisor of `p - 1` and `p` divides their difference `1` — yields
// the corollary.  The cancellation step is the library theorem
// `cancel_coprime`.
//
// The corollary is the `k = 2` instance of the general fact that
//     (p - k)! · (p - 1)(p - 2)...(p - k + 1) ≡ 1  (mod p),
// and it is the step that shows `(p - 1)!` is the smallest factorial
// congruent to `-1` modulo `p`.

/// `(p - 1) - 1 = p - 2` for `p >= 2`: subtracting one twice subtracts two.
theorem wilson_sub_sub(p: Nat) {
    Nat.1 < p implies (p - Nat.1) - Nat.1 = p - Nat.2
} by {
    if Nat.1 < p {
        lt_suc(Nat.0)
        Nat.0 < Nat.1
        lt_trans(Nat.0, Nat.1, p)
        Nat.0 < p
        let m: Nat satisfy { m.suc = p }
        m.suc = p
        suc_sub_one(m)
        m.suc - Nat.1 = m
        p - Nat.1 = m
        sub_pos(p, Nat.1)
        p - Nat.1 > Nat.0
        Nat.0 < m
        let l: Nat satisfy { l.suc = m }
        l.suc = m
        suc_sub_one(l)
        l.suc - Nat.1 = l
        m - Nat.1 = l
        (p - Nat.1) - Nat.1 = l
        l.suc.suc = m.suc
        m.suc = p
        l.suc.suc = p
        l + Nat.2 = l.suc.suc
        l + Nat.2 = p
        add_imp_sub(l, Nat.2, p)
        p - Nat.2 = l
        (p - Nat.1) - Nat.1 = p - Nat.2
    }
}

/// `p - 1` is coprime to the prime `p`: any common divisor of `p - 1` and
/// `p` divides their difference `1`, and one is the only divisor of one.
theorem wilson_pred_coprime(p: Nat) {
    p.is_prime implies (p - Nat.1).coprime(p)
} by {
    if p.is_prime {
        Nat.1 < p
        // d = gcd(p - 1, p) divides both arguments.
        gcd_divides_left(p - Nat.1, p)
        (p - Nat.1).gcd(p).divides(p - Nat.1)
        gcd_divides_right(p - Nat.1, p)
        (p - Nat.1).gcd(p).divides(p)
        // A common divisor divides the difference: d | p - (p - 1) = 1.
        divides_sub(p, p - Nat.1, (p - Nat.1).gcd(p))
        (p - Nat.1).gcd(p).divides(p - (p - Nat.1))
        add_sub(p, Nat.1)
        Nat.1 <= p
        p - Nat.1 + Nat.1 = p
        add_imp_sub(p - Nat.1, Nat.1, p)
        p - (p - Nat.1) = Nat.1
        (p - Nat.1).gcd(p).divides(Nat.1)
        nat_divides_one_imp_one((p - Nat.1).gcd(p))
        (p - Nat.1).gcd(p) = Nat.1
        (p - Nat.1).coprime(p)
    }
}

/// Wilson's corollary: for a prime `p`, `(p - 2)! ≡ 1 (mod p)`.
theorem theorems1000_wilson_corollary(p: Nat) {
    p.is_prime implies (p - Nat.2).factorial.congr_mod(Nat.1, p)
} by {
    if p.is_prime {
        Nat.1 < p
        // Wilson's theorem: (p - 1)! ≡ p - 1 (mod p).
        prime_imp_wilson_factorial_congr(p)
        (p - Nat.1).factorial.congr_mod(p - Nat.1, p)
        // Split the factorial: (p - 1)! = (p - 1) * (p - 2)!.
        sub_pos(p, Nat.1)
        p - Nat.1 > Nat.0
        let q: Nat satisfy { q.suc = p - Nat.1 }
        q.suc = p - Nat.1
        factorial_step(q)
        q.suc.factorial = q.suc * q.factorial
        (p - Nat.1).factorial = (p - Nat.1) * q.factorial
        add_imp_sub(q, Nat.1, p - Nat.1)
        (p - Nat.1) - Nat.1 = q
        wilson_sub_sub(p)
        (p - Nat.1) - Nat.1 = p - Nat.2
        q = p - Nat.2
        (p - Nat.1).factorial = (p - Nat.1) * (p - Nat.2).factorial
        ((p - Nat.1) * (p - Nat.2).factorial).congr_mod(p - Nat.1, p)
        // Rewrite the right side as (p - 1) * 1.
        mul_one_right(p - Nat.1)
        (p - Nat.1) * Nat.1 = p - Nat.1
        ((p - Nat.1) * (p - Nat.2).factorial).congr_mod((p - Nat.1) * Nat.1, p)
        // Cancel the factor p - 1, which is coprime to p.
        wilson_pred_coprime(p)
        (p - Nat.1).coprime(p)
        cancel_coprime(p - Nat.1, p, (p - Nat.2).factorial, Nat.1)
        (p - Nat.2).factorial.congr_mod(Nat.1, p)
    }
}
