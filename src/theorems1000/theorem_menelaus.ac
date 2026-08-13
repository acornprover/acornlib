from pair import Pair
from real import Real, mul_left_cancel
from top100 import r2_add, r2_sub
from theorems1000.r2_helpers import r2_smul, r2_cross, r2_affine_diff_bc, r2_affine_diff_ca,
    r2_cross_add_left, r2_cross_add_right, r2_cross_smul_left, r2_cross_smul_right,
    r2_cross_swap, r2_cross_self_zero, r2_cross_sub_left, r2_cross_sub_right

numerals Real

// Menelaus's theorem: three points on the three sides of a triangle are
// collinear exactly when the product of the three directed ratios in which
// they divide the sides equals minus one.
//
// Points on the sides are written as affine combinations: `d` on `bc` with
// parameter `t1` is `d = (1 - t1) b + t1 c`, and the directed ratio
// `BD/DC = t1/(1 - t1)`; likewise `e = (1 - t2) c + t2 a` on `ca` and
// `f = (1 - t3) a + t3 b` on `ab`.  Collinearity of `d`, `e`, `f` is
// `cross(d - f, e - f) = 0`, and Menelaus's theorem reads
//     t1 t2 t3 + (1 - t1)(1 - t2)(1 - t3) = 0,
// which is the cross-multiplied form of `(BD/DC)(CE/EA)(AF/FB) = -1`.
//
// Proof: write `u = b - a` and `v = c - a`.  From the affine combinations,
//     d - f = (1 - t1 - t3) u + t1 v,
//     e - f = (1 - t2) v - t3 u,
// and expanding the cross product with the bilinearity lemmas gives
//     cross(d - f, e - f) = ((1 - t1 - t3)(1 - t2) + t1 t3) * cross(u, v).
// The collinearity hypothesis makes the left side zero, and the
// non-degeneracy hypothesis `cross(u, v) != 0` allows cancelling
// `cross(u, v)` by the field zero-product law, so
//     (1 - t1 - t3)(1 - t2) + t1 t3 = 0.
// Finally the identity
//     t1 t2 t3 + (1 - t1)(1 - t2)(1 - t3)
//         = (1 - t1 - t3)(1 - t2) + t1 t3
// identifies this with the cross-multiplied ratio condition, i.e.
// `(BD/DC)(CE/EA)(AF/FB) = -1`.

/// `(1 - t1)(1 - t3) - (1 - t1 - t3) = t1 t3`: expanding the product leaves
/// the difference of the two one-minus terms.
theorem menelaus_diff_expand(t1: Real, t3: Real) {
    (Real.1 - t1) * (Real.1 - t3) - (Real.1 - t1 - t3) = t1 * t3
} by {
    (Real.1 - t1) * (Real.1 - t3) =
        (Real.1 - t1) * Real.1 - (Real.1 - t1) * t3
    (Real.1 - t1) * Real.1 = Real.1 - t1
    (Real.1 - t1) * t3 = t3 - t1 * t3
    (Real.1 - t1) * (Real.1 - t3) = (Real.1 - t1) - (t3 - t1 * t3)
    (Real.1 - t1) - (t3 - t1 * t3) = (Real.1 - t1) + -(t3 - t1 * t3)
    -(t3 - t1 * t3) = -t3 + t1 * t3
    (Real.1 - t1) + (-t3 + t1 * t3) = Real.1 - t1 - t3 + t1 * t3
    (Real.1 - t1) * (Real.1 - t3) = Real.1 - t1 - t3 + t1 * t3
    (Real.1 - t1 - t3) + t1 * t3 = Real.1 - t1 - t3 + t1 * t3
    (Real.1 - t1 - t3 + t1 * t3) - (Real.1 - t1 - t3) = t1 * t3
    (Real.1 - t1) * (Real.1 - t3) - (Real.1 - t1 - t3) = t1 * t3
}

/// Factoring `(1 - t2)` out of the difference of the two triple products:
/// `(1 - t1)(1 - t2)(1 - t3) - (1 - t1 - t3)(1 - t2) = (1 - t2) t1 t3`.
theorem menelaus_factor(t1: Real, t2: Real, t3: Real) {
    (Real.1 - t1) * (Real.1 - t2) * (Real.1 - t3) -
        (Real.1 - t1 - t3) * (Real.1 - t2) =
        (Real.1 - t2) * (t1 * t3)
} by {
    (Real.1 - t1) * (Real.1 - t2) * (Real.1 - t3) =
        (Real.1 - t2) * (Real.1 - t1) * (Real.1 - t3)
    (Real.1 - t1 - t3) * (Real.1 - t2) =
        (Real.1 - t2) * (Real.1 - t1 - t3)
    (Real.1 - t2) * (Real.1 - t1) * (Real.1 - t3) -
        (Real.1 - t2) * (Real.1 - t1 - t3) =
        (Real.1 - t2) * ((Real.1 - t1) * (Real.1 - t3) - (Real.1 - t1 - t3))
    menelaus_diff_expand(t1, t3)
    (Real.1 - t1) * (Real.1 - t3) - (Real.1 - t1 - t3) = t1 * t3
    (Real.1 - t2) * ((Real.1 - t1) * (Real.1 - t3) - (Real.1 - t1 - t3)) =
        (Real.1 - t2) * (t1 * t3)
    (Real.1 - t1) * (Real.1 - t2) * (Real.1 - t3) -
        (Real.1 - t1 - t3) * (Real.1 - t2) =
        (Real.1 - t2) * (t1 * t3)
}

/// Distributing `(1 - t2)` over `t1 t3`:
/// `(1 - t2)(t1 t3) = t1 t3 - t1 t2 t3`.
theorem menelaus_mul_distrib(t1: Real, t2: Real, t3: Real) {
    (Real.1 - t2) * (t1 * t3) = t1 * t3 - t1 * t2 * t3
} by {
    (Real.1 - t2) * (t1 * t3) =
        Real.1 * (t1 * t3) - t2 * (t1 * t3)
    Real.1 * (t1 * t3) = t1 * t3
    t2 * (t1 * t3) = t1 * t2 * t3
    (Real.1 - t2) * (t1 * t3) = t1 * t3 - t1 * t2 * t3
}

/// The regrouped difference vanishes:
/// `t1 t2 t3 + ((1 - t1)(1 - t2)(1 - t3) - (1 - t1 - t3)(1 - t2)) - t1 t3 = 0`.
theorem menelaus_diff_zero(t1: Real, t2: Real, t3: Real) {
    t1 * t2 * t3 +
        ((Real.1 - t1) * (Real.1 - t2) * (Real.1 - t3) -
            (Real.1 - t1 - t3) * (Real.1 - t2)) - t1 * t3 = Real.0
} by {
    menelaus_factor(t1, t2, t3)
    (Real.1 - t1) * (Real.1 - t2) * (Real.1 - t3) -
        (Real.1 - t1 - t3) * (Real.1 - t2) =
        (Real.1 - t2) * (t1 * t3)
    menelaus_mul_distrib(t1, t2, t3)
    (Real.1 - t2) * (t1 * t3) = t1 * t3 - t1 * t2 * t3
    t1 * t2 * t3 + ((Real.1 - t1) * (Real.1 - t2) * (Real.1 - t3) -
        (Real.1 - t1 - t3) * (Real.1 - t2)) - t1 * t3 =
        t1 * t2 * t3 + ((Real.1 - t2) * (t1 * t3)) - t1 * t3
    t1 * t2 * t3 + (t1 * t3 - t1 * t2 * t3) - t1 * t3 =
        t1 * t2 * t3 + t1 * t3 - t1 * t2 * t3 - t1 * t3
    t1 * t2 * t3 + t1 * t3 - t1 * t2 * t3 - t1 * t3 = Real.0
    t1 * t2 * t3 + (t1 * t3 - t1 * t2 * t3) - t1 * t3 = Real.0
}

/// Regrouping a flat difference: `x + y - z - w = x + (y - z) - w`.
theorem menelaus_flat_group(x: Real, y: Real, z: Real, w: Real) {
    x + y - z - w = x + (y - z) - w
}

/// The first bilinear piece of the cross expansion: with `u = b - a` and
/// `v = c - a`,
/// `cross((1 - t1 - t3) u, (1 - t2) v - t3 u) =
///      (1 - t1 - t3)((1 - t2) cross(u, v))`.
theorem menelaus_first_piece(u: Pair[Real, Real], v: Pair[Real, Real],
    t1: Real, t2: Real, t3: Real) {
    r2_cross(
        r2_smul(Real.1 - t1 - t3, u),
        r2_sub(r2_smul(Real.1 - t2, v), r2_smul(t3, u))
    ) =
    (Real.1 - t1 - t3) * ((Real.1 - t2) * r2_cross(u, v))
} by {
    r2_cross_sub_right(
        r2_smul(Real.1 - t1 - t3, u),
        r2_smul(Real.1 - t2, v),
        r2_smul(t3, u)
    )
    r2_cross(
        r2_smul(Real.1 - t1 - t3, u),
        r2_sub(r2_smul(Real.1 - t2, v), r2_smul(t3, u))
    ) =
    r2_cross(r2_smul(Real.1 - t1 - t3, u), r2_smul(Real.1 - t2, v)) -
        r2_cross(r2_smul(Real.1 - t1 - t3, u), r2_smul(t3, u))
    r2_cross_smul_left(Real.1 - t1 - t3, u, r2_smul(Real.1 - t2, v))
    r2_cross(r2_smul(Real.1 - t1 - t3, u), r2_smul(Real.1 - t2, v)) =
        (Real.1 - t1 - t3) * r2_cross(u, r2_smul(Real.1 - t2, v))
    r2_cross_smul_right(Real.1 - t2, v, u)
    r2_cross(u, r2_smul(Real.1 - t2, v)) =
        (Real.1 - t2) * r2_cross(u, v)
    r2_cross(r2_smul(Real.1 - t1 - t3, u), r2_smul(Real.1 - t2, v)) =
        (Real.1 - t1 - t3) * ((Real.1 - t2) * r2_cross(u, v))
    r2_cross_smul_left(Real.1 - t1 - t3, u, r2_smul(t3, u))
    r2_cross(r2_smul(Real.1 - t1 - t3, u), r2_smul(t3, u)) =
        (Real.1 - t1 - t3) * r2_cross(u, r2_smul(t3, u))
    r2_cross_smul_right(t3, u, u)
    r2_cross(u, r2_smul(t3, u)) = t3 * r2_cross(u, u)
    r2_cross_self_zero(u)
    r2_cross(u, u) = Real.0
    t3 * r2_cross(u, u) = Real.0
    r2_cross(u, r2_smul(t3, u)) = Real.0
    (Real.1 - t1 - t3) * r2_cross(u, r2_smul(t3, u)) = Real.0
    r2_cross(r2_smul(Real.1 - t1 - t3, u), r2_smul(t3, u)) = Real.0
    r2_cross(
        r2_smul(Real.1 - t1 - t3, u),
        r2_sub(r2_smul(Real.1 - t2, v), r2_smul(t3, u))
    ) =
    (Real.1 - t1 - t3) * ((Real.1 - t2) * r2_cross(u, v)) - Real.0
    (Real.1 - t1 - t3) * ((Real.1 - t2) * r2_cross(u, v)) - Real.0 =
        (Real.1 - t1 - t3) * ((Real.1 - t2) * r2_cross(u, v))
    r2_cross(
        r2_smul(Real.1 - t1 - t3, u),
        r2_sub(r2_smul(Real.1 - t2, v), r2_smul(t3, u))
    ) =
    (Real.1 - t1 - t3) * ((Real.1 - t2) * r2_cross(u, v))
}

/// The second bilinear piece of the cross expansion:
/// `cross(t1 v, (1 - t2) v - t3 u) = t1 (t3 cross(u, v))`.
theorem menelaus_second_piece(u: Pair[Real, Real], v: Pair[Real, Real],
    t1: Real, t2: Real, t3: Real) {
    r2_cross(
        r2_smul(t1, v),
        r2_sub(r2_smul(Real.1 - t2, v), r2_smul(t3, u))
    ) = t1 * (t3 * r2_cross(u, v))
} by {
    r2_cross_sub_right(r2_smul(t1, v), r2_smul(Real.1 - t2, v), r2_smul(t3, u))
    r2_cross(
        r2_smul(t1, v),
        r2_sub(r2_smul(Real.1 - t2, v), r2_smul(t3, u))
    ) =
    r2_cross(r2_smul(t1, v), r2_smul(Real.1 - t2, v)) -
        r2_cross(r2_smul(t1, v), r2_smul(t3, u))
    r2_cross_smul_left(t1, v, r2_smul(Real.1 - t2, v))
    r2_cross(r2_smul(t1, v), r2_smul(Real.1 - t2, v)) =
        t1 * r2_cross(v, r2_smul(Real.1 - t2, v))
    r2_cross_smul_right(Real.1 - t2, v, v)
    r2_cross(v, r2_smul(Real.1 - t2, v)) =
        (Real.1 - t2) * r2_cross(v, v)
    r2_cross_self_zero(v)
    r2_cross(v, v) = Real.0
    (Real.1 - t2) * r2_cross(v, v) = Real.0
    r2_cross(v, r2_smul(Real.1 - t2, v)) = Real.0
    t1 * r2_cross(v, r2_smul(Real.1 - t2, v)) = Real.0
    r2_cross(r2_smul(t1, v), r2_smul(Real.1 - t2, v)) = Real.0
    r2_cross_smul_left(t1, v, r2_smul(t3, u))
    r2_cross(r2_smul(t1, v), r2_smul(t3, u)) =
        t1 * r2_cross(v, r2_smul(t3, u))
    r2_cross_smul_right(t3, u, v)
    r2_cross(v, r2_smul(t3, u)) = t3 * r2_cross(v, u)
    r2_cross(r2_smul(t1, v), r2_smul(t3, u)) = t1 * (t3 * r2_cross(v, u))
    r2_cross_swap(u, v)
    r2_cross(u, v) = -r2_cross(v, u)
    r2_cross(
        r2_smul(t1, v),
        r2_sub(r2_smul(Real.1 - t2, v), r2_smul(t3, u))
    ) =
    Real.0 - t1 * (t3 * r2_cross(v, u))
    Real.0 - t1 * (t3 * r2_cross(v, u)) =
        t1 * (t3 * r2_cross(u, v))
    r2_cross(
        r2_smul(t1, v),
        r2_sub(r2_smul(Real.1 - t2, v), r2_smul(t3, u))
    ) = t1 * (t3 * r2_cross(u, v))
}

/// The ring identity behind Menelaus:
/// `t1 t2 t3 + (1 - t1)(1 - t2)(1 - t3) = (1 - t1 - t3)(1 - t2) + t1 t3`.
theorem menelaus_ring_identity(t1: Real, t2: Real, t3: Real) {
    t1 * t2 * t3 + (Real.1 - t1) * (Real.1 - t2) * (Real.1 - t3) =
        (Real.1 - t1 - t3) * (Real.1 - t2) + t1 * t3
} by {
    menelaus_diff_zero(t1, t2, t3)
    t1 * t2 * t3 +
        ((Real.1 - t1) * (Real.1 - t2) * (Real.1 - t3) -
            (Real.1 - t1 - t3) * (Real.1 - t2)) - t1 * t3 = Real.0
    menelaus_flat_group(t1 * t2 * t3,
        (Real.1 - t1) * (Real.1 - t2) * (Real.1 - t3),
        (Real.1 - t1 - t3) * (Real.1 - t2), t1 * t3)
    t1 * t2 * t3 + (Real.1 - t1) * (Real.1 - t2) * (Real.1 - t3) -
        (Real.1 - t1 - t3) * (Real.1 - t2) - t1 * t3 =
        t1 * t2 * t3 +
            ((Real.1 - t1) * (Real.1 - t2) * (Real.1 - t3) -
                (Real.1 - t1 - t3) * (Real.1 - t2)) - t1 * t3
    t1 * t2 * t3 + (Real.1 - t1) * (Real.1 - t2) * (Real.1 - t3) -
        (Real.1 - t1 - t3) * (Real.1 - t2) - t1 * t3 = Real.0
    // the left-hand side minus the right-hand side is the flat form
    t1 * t2 * t3 + (Real.1 - t1) * (Real.1 - t2) * (Real.1 - t3) -
        ((Real.1 - t1 - t3) * (Real.1 - t2) + t1 * t3) =
        t1 * t2 * t3 + (Real.1 - t1) * (Real.1 - t2) * (Real.1 - t3) -
            (Real.1 - t1 - t3) * (Real.1 - t2) - t1 * t3
    t1 * t2 * t3 + (Real.1 - t1) * (Real.1 - t2) * (Real.1 - t3) -
        ((Real.1 - t1 - t3) * (Real.1 - t2) + t1 * t3) = Real.0
    // a difference of zero gives equality
    t1 * t2 * t3 + (Real.1 - t1) * (Real.1 - t2) * (Real.1 - t3) =
        (Real.1 - t1 - t3) * (Real.1 - t2) + t1 * t3
}

// Menelaus's theorem: if the points `d` on `bc`, `e` on `ca` and `f` on
// `ab` are collinear, then the product of the directed ratios equals minus
// one: `t1 t2 t3 + (1 - t1)(1 - t2)(1 - t3) = 0`.

/// Menelaus's theorem: for `d = (1 - t1) b + t1 c` on `bc`,
/// `e = (1 - t2) c + t2 a` on `ca` and `f = (1 - t3) a + t3 b` on `ab`,
/// the collinearity of `d`, `e`, `f` implies
/// `t1 t2 t3 + (1 - t1)(1 - t2)(1 - t3) = 0` (the cross-multiplied ratio
/// condition `(BD/DC)(CE/EA)(AF/FB) = -1`).
theorem theorems1000_menelaus(
    a: Pair[Real, Real], b: Pair[Real, Real], c: Pair[Real, Real],
    d: Pair[Real, Real], e: Pair[Real, Real], f: Pair[Real, Real],
    t1: Real, t2: Real, t3: Real
) {
    r2_add(r2_smul(Real.1 - t1, b), r2_smul(t1, c)) = d and
    r2_add(r2_smul(Real.1 - t2, c), r2_smul(t2, a)) = e and
    r2_add(r2_smul(Real.1 - t3, a), r2_smul(t3, b)) = f and
    r2_cross(r2_sub(d, f), r2_sub(e, f)) = Real.0 and
    r2_cross(r2_sub(b, a), r2_sub(c, a)) != Real.0
    implies
    t1 * t2 * t3 + (Real.1 - t1) * (Real.1 - t2) * (Real.1 - t3) = Real.0
} by {
    if r2_add(r2_smul(Real.1 - t1, b), r2_smul(t1, c)) = d and
        r2_add(r2_smul(Real.1 - t2, c), r2_smul(t2, a)) = e and
        r2_add(r2_smul(Real.1 - t3, a), r2_smul(t3, b)) = f and
        r2_cross(r2_sub(d, f), r2_sub(e, f)) = Real.0 and
        r2_cross(r2_sub(b, a), r2_sub(c, a)) != Real.0 {
        r2_add(r2_smul(Real.1 - t1, b), r2_smul(t1, c)) = d
        r2_add(r2_smul(Real.1 - t2, c), r2_smul(t2, a)) = e
        r2_add(r2_smul(Real.1 - t3, a), r2_smul(t3, b)) = f
        r2_cross(r2_sub(d, f), r2_sub(e, f)) = Real.0
        r2_cross(r2_sub(b, a), r2_sub(c, a)) != Real.0
        // d - f = (1 - t1 - t3)(b - a) + t1 (c - a)
        r2_affine_diff_bc(a, b, c, t1, t3)
        r2_sub(
            r2_add(r2_smul(Real.1 - t1, b), r2_smul(t1, c)),
            r2_add(r2_smul(Real.1 - t3, a), r2_smul(t3, b))
        ) =
        r2_add(
            r2_smul(Real.1 - t1 - t3, r2_sub(b, a)),
            r2_smul(t1, r2_sub(c, a))
        )
        r2_sub(d, f) =
        r2_add(
            r2_smul(Real.1 - t1 - t3, r2_sub(b, a)),
            r2_smul(t1, r2_sub(c, a))
        )
        // e - f = (1 - t2)(c - a) - t3 (b - a)
        r2_affine_diff_ca(a, b, c, t2, t3)
        r2_sub(
            r2_add(r2_smul(Real.1 - t2, c), r2_smul(t2, a)),
            r2_add(r2_smul(Real.1 - t3, a), r2_smul(t3, b))
        ) =
        r2_sub(
            r2_smul(Real.1 - t2, r2_sub(c, a)),
            r2_smul(t3, r2_sub(b, a))
        )
        r2_sub(e, f) =
        r2_sub(
            r2_smul(Real.1 - t2, r2_sub(c, a)),
            r2_smul(t3, r2_sub(b, a))
        )
        // expand cross(d - f, e - f) = cross((1-t1-t3)u + t1v, (1-t2)v - t3u)
        r2_cross(r2_sub(d, f), r2_sub(e, f)) =
        r2_cross(
            r2_add(
                r2_smul(Real.1 - t1 - t3, r2_sub(b, a)),
                r2_smul(t1, r2_sub(c, a))
            ),
            r2_sub(
                r2_smul(Real.1 - t2, r2_sub(c, a)),
                r2_smul(t3, r2_sub(b, a))
            )
        )
        r2_cross_add_left(
            r2_smul(Real.1 - t1 - t3, r2_sub(b, a)),
            r2_smul(t1, r2_sub(c, a)),
            r2_sub(
                r2_smul(Real.1 - t2, r2_sub(c, a)),
                r2_smul(t3, r2_sub(b, a))
            )
        )
        r2_cross(
            r2_add(
                r2_smul(Real.1 - t1 - t3, r2_sub(b, a)),
                r2_smul(t1, r2_sub(c, a))
            ),
            r2_sub(
                r2_smul(Real.1 - t2, r2_sub(c, a)),
                r2_smul(t3, r2_sub(b, a))
            )
        ) =
        r2_cross(
            r2_smul(Real.1 - t1 - t3, r2_sub(b, a)),
            r2_sub(
                r2_smul(Real.1 - t2, r2_sub(c, a)),
                r2_smul(t3, r2_sub(b, a))
            )
        ) +
        r2_cross(
            r2_smul(t1, r2_sub(c, a)),
            r2_sub(
                r2_smul(Real.1 - t2, r2_sub(c, a)),
                r2_smul(t3, r2_sub(b, a))
            )
        )
        // first piece
        menelaus_first_piece(r2_sub(b, a), r2_sub(c, a), t1, t2, t3)
        r2_cross(
            r2_smul(Real.1 - t1 - t3, r2_sub(b, a)),
            r2_sub(
                r2_smul(Real.1 - t2, r2_sub(c, a)),
                r2_smul(t3, r2_sub(b, a))
            )
        ) =
        (Real.1 - t1 - t3) *
            ((Real.1 - t2) * r2_cross(r2_sub(b, a), r2_sub(c, a)))
        r2_cross(r2_sub(d, f), r2_sub(e, f)) =
        (Real.1 - t1 - t3) *
            ((Real.1 - t2) * r2_cross(r2_sub(b, a), r2_sub(c, a))) +
        r2_cross(
            r2_smul(t1, r2_sub(c, a)),
            r2_sub(
                r2_smul(Real.1 - t2, r2_sub(c, a)),
                r2_smul(t3, r2_sub(b, a))
            )
        )
        // second piece
        menelaus_second_piece(r2_sub(b, a), r2_sub(c, a), t1, t2, t3)
        r2_cross(
            r2_smul(t1, r2_sub(c, a)),
            r2_sub(
                r2_smul(Real.1 - t2, r2_sub(c, a)),
                r2_smul(t3, r2_sub(b, a))
            )
        ) =
        t1 * (t3 * r2_cross(r2_sub(b, a), r2_sub(c, a)))
        r2_cross(r2_sub(d, f), r2_sub(e, f)) =
        (Real.1 - t1 - t3) *
            ((Real.1 - t2) * r2_cross(r2_sub(b, a), r2_sub(c, a))) +
        t1 * (t3 * r2_cross(r2_sub(b, a), r2_sub(c, a)))
        // factor the common cross
        (Real.1 - t1 - t3) *
            ((Real.1 - t2) * r2_cross(r2_sub(b, a), r2_sub(c, a))) +
        t1 * (t3 * r2_cross(r2_sub(b, a), r2_sub(c, a))) =
            ((Real.1 - t1 - t3) * (Real.1 - t2) + t1 * t3) *
                r2_cross(r2_sub(b, a), r2_sub(c, a))
        r2_cross(r2_sub(d, f), r2_sub(e, f)) =
        ((Real.1 - t1 - t3) * (Real.1 - t2) + t1 * t3) *
            r2_cross(r2_sub(b, a), r2_sub(c, a))
        // hypothesis: cross(d-f, e-f) = 0
        ((Real.1 - t1 - t3) * (Real.1 - t2) + t1 * t3) *
            r2_cross(r2_sub(b, a), r2_sub(c, a)) = Real.0
        // cancel the nonzero cross
        Real.0 =
            r2_cross(r2_sub(b, a), r2_sub(c, a)) *
                ((Real.1 - t1 - t3) * (Real.1 - t2) + t1 * t3)
        mul_left_cancel(
            Real.0,
            r2_cross(r2_sub(b, a), r2_sub(c, a)),
            (Real.1 - t1 - t3) * (Real.1 - t2) + t1 * t3)
        Real.0 / r2_cross(r2_sub(b, a), r2_sub(c, a)) =
            (Real.1 - t1 - t3) * (Real.1 - t2) + t1 * t3
        Real.0 / r2_cross(r2_sub(b, a), r2_sub(c, a)) = Real.0
        (Real.1 - t1 - t3) * (Real.1 - t2) + t1 * t3 = Real.0
        // ring identity
        menelaus_ring_identity(t1, t2, t3)
        t1 * t2 * t3 + (Real.1 - t1) * (Real.1 - t2) * (Real.1 - t3) =
            (Real.1 - t1 - t3) * (Real.1 - t2) + t1 * t3
        t1 * t2 * t3 + (Real.1 - t1) * (Real.1 - t2) * (Real.1 - t3) = Real.0
    }
}
