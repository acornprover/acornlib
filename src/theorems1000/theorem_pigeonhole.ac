from nat import Nat, lt_not_ref, lte_and_lt
from list import List, map, length_range, range_contains_of_lt, lt_of_range_contains,
    range_is_unique, map_length, map_contains, unique_is_smallest_containing_list,
    unique_length
from data.list.list_pigeonhole import pigeonhole_unique_map_in

numerals Nat

// The pigeonhole principle (also called the Dirichlet box principle, after
// Peter Gustav Lejeune Dirichlet, who used it in 1834 in his proof of
// Dirichlet's approximation theorem): if more than `n` objects are placed
// into `n` boxes, then some box receives at least two objects.  The form
// proved here is the counting formulation over the naturals: if `m > n`
// elements are distributed into the `n` boxes `0, 1, ..., n - 1` by a
// function `f` (so every value `f(i)` with `i < m` lies below `n`), then two
// distinct elements `x, y < m` land in the same box, i.e. `f(x) = f(y)`.
//
// The principle is the foundation of many existence arguments: among `n + 1`
// integers, two are congruent modulo `n`; in any group of 367 people, two
// share a birthday; every sequence of `(r - 1)(s - 1) + 1` distinct real
// numbers contains an increasing subsequence of length `r` or a decreasing
// subsequence of length `s` (the Erdős–Szekeres theorem); and so on.
//
// Proof: the library list machinery counts the distinct values of `f` on
// `[0, m)`.  Since every value lies below `n`, the list `map(m.range, f)`
// contains only elements of `n.range`, so its deduplicated image has length
// at most `n` (`unique_is_smallest_containing_list`).  But the source list
// `m.range` has length `m > n`, so the image is not unique, and
// `pigeonhole_unique_map_in` then names two members `x, y` of `m.range` with
// `x != y` and `f(x) = f(y)`.

/// If `m.range.unique` is shorter than `m.range`, the list has a duplicate.
theorem unique_length_lt_imp_not_unique[T](list: List[T]) {
    list.unique.length < list.length implies not list.is_unique
} by {
    if list.unique.length < list.length {
        if list.is_unique {
            list.unique = list
            list.unique.length = list.length
            list.length < list.length
            lt_not_ref(list.length)
            false
        }
        not list.is_unique
    }
}

/// The pigeonhole principle: if `m > n` values in `[0, m)` all lie below
/// `n`, then two distinct values are equal.
theorem theorems1000_pigeonhole(n: Nat, m: Nat, f: Nat -> Nat) {
    n < m and (forall(i: Nat) { i < m implies f(i) < n }) implies
        exists(x: Nat, y: Nat) { x < m and y < m and x != y and f(x) = f(y) }
} by {
    if n < m and (forall(i: Nat) { i < m implies f(i) < n }) {
        // Every element of the image list map(m.range, f) lies in [0, n).
        forall(x: Nat) {
            if map(m.range, f).contains(x) {
                map_contains[Nat, Nat](m.range, f, x)
                let (y: Nat) satisfy { m.range.contains(y) and f(y) = x }
                lt_of_range_contains(m, y)
                y < m
                f(y) < n
                x < n
                range_contains_of_lt(n, x)
                n.range.contains(x)
            }
        }
        // The deduplicated image of [0, m) fits inside [0, n).
        unique_is_smallest_containing_list[Nat](map(m.range, f), n.range)
        map(m.range, f).unique.length <= n.range.length
        length_range(m)
        m.range.length = m
        length_range(n)
        n.range.length = n
        map(m.range, f).unique.length <= n
        lte_and_lt(map(m.range, f).unique.length, n, m)
        map(m.range, f).unique.length < m
        // ... while the source list [0, m) has length m, so the image of
        // the unique list is not unique.
        map_length[Nat, Nat](m.range, f)
        map(m.range, f).length = m.range.length
        map(m.range, f).unique.length < map(m.range, f).length
        unique_length_lt_imp_not_unique[Nat](map(m.range, f))
        not map(m.range, f).is_unique
        // A non-unique image of a unique source forces a collision inside
        // [0, m).
        range_is_unique(m)
        m.range.is_unique
        pigeonhole_unique_map_in[Nat, Nat](m.range, f)
        exists(x: Nat, y: Nat) {
            m.range.contains(x) and m.range.contains(y) and x != y and f(x) = f(y)
        }
        let (x: Nat, y: Nat) satisfy {
            m.range.contains(x) and m.range.contains(y) and x != y and f(x) = f(y)
        }
        lt_of_range_contains(m, x)
        x < m
        lt_of_range_contains(m, y)
        y < m
        exists(x2: Nat, y2: Nat) { x2 < m and y2 < m and x2 != y2 and f(x2) = f(y2) }
    }
}

/// The `n + 1` pigeons into `n` boxes form: among the values
/// `f(0), ..., f(n)`, all lying below `n`, two are equal.
theorem theorems1000_pigeonhole_suc(n: Nat, f: Nat -> Nat) {
    (forall(i: Nat) { i < n + Nat.1 implies f(i) < n }) implies
        exists(x: Nat, y: Nat) { x < n + Nat.1 and y < n + Nat.1 and x != y and f(x) = f(y) }
} by {
    if forall(i: Nat) { i < n + Nat.1 implies f(i) < n } {
        n < n + Nat.1
        theorems1000_pigeonhole(n, n + Nat.1, f)
        exists(x: Nat, y: Nat) { x < n + Nat.1 and y < n + Nat.1 and x != y and f(x) = f(y) }
    }
}
