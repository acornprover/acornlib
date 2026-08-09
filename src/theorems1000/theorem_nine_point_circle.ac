from pair import Pair, pair_ext
from real import Real
from top100 import r2_add, r2_neg, r2_sub, r2_norm_sq
from theorems1000.r2_helpers import r2_double_sub_distrib, r2_sub_add_pair_distrib,
    r2_sub_eq_add_neg, r2_add_comm, r2_add_three_rotate, r2_sub_add_right_cancel,
    r2_sub_reverse_neg, r2_norm_sq_neg, r2_norm_sq_double, r2_add_assoc

// The nine-point circle: for a triangle, the midpoints of the three sides,
// the feet of the three altitudes and the midpoints of the segments from the
// orthocenter to the vertices all lie on a single circle, whose center is
// the midpoint of the circumcenter and the orthocenter and whose radius is
// half the circumradius.  (The circle is the subject of Feuerbach's theorem,
// which asserts that it is tangent to the incircle and the excircles.)
//
// With `o` the circumcenter (`|a-o|^2 = |b-o|^2 = |c-o|^2 = radius_sq`), the
// candidate orthocenter `h = a + b + c - 2o`, the nine-point center `n` with
// `2n = o + h`, and a side midpoint `m` with `2m = a + b`, the identity
// `2 (m - n) = o - c` shows `4 |m - n|^2 = |o - c|^2 = radius_sq`, so the
// three side midpoints lie on the circle with center `n` and squared radius
// `radius_sq / 4`.  Likewise, for an Euler point `e` (midpoint of `a` and
// `h`, `2e = a + h`), `2 (e - n) = a - o`, so `4 |e - n|^2 = radius_sq` as
// well.  These two families are proved below; the third family (the feet of
// the altitudes) requires the explicit computation of the foot as an affine
// combination of the side endpoints and is recorded as a statement with a
// proof sketch.

// ---------------------------------------------------------------------------
// Small vector-algebra helpers.
// ---------------------------------------------------------------------------

/// A point minus itself is the origin.
theorem r2_sub_self_zero(a: Pair[Real, Real]) {
    r2_sub(a, a) = Pair.new(Real.0, Real.0)
} by {
    r2_sub(a, a).first = a.first - a.first
    a.first - a.first = Real.0
    r2_sub(a, a).second = a.second - a.second
    a.second - a.second = Real.0
    pair_ext(r2_sub(a, a), Pair.new(Real.0, Real.0))
}

/// Adding the origin leaves a point unchanged.
theorem r2_add_zero_right(a: Pair[Real, Real]) {
    r2_add(a, Pair.new(Real.0, Real.0)) = a
} by {
    r2_add(a, Pair.new(Real.0, Real.0)).first =
        a.first + Pair.new(Real.0, Real.0).first
    Pair.new(Real.0, Real.0).first = Real.0
    r2_add(a, Pair.new(Real.0, Real.0)).first = a.first + Real.0
    a.first + Real.0 = a.first
    r2_add(a, Pair.new(Real.0, Real.0)).second =
        a.second + Pair.new(Real.0, Real.0).second
    Pair.new(Real.0, Real.0).second = Real.0
    r2_add(a, Pair.new(Real.0, Real.0)).second = a.second + Real.0
    a.second + Real.0 = a.second
    pair_ext(r2_add(a, Pair.new(Real.0, Real.0)), a)
}

/// Subtracting the origin leaves a point unchanged.
theorem r2_sub_zero_right(a: Pair[Real, Real]) {
    r2_sub(a, Pair.new(Real.0, Real.0)) = a
} by {
    r2_sub(a, Pair.new(Real.0, Real.0)).first =
        a.first - Pair.new(Real.0, Real.0).first
    Pair.new(Real.0, Real.0).first = Real.0
    r2_sub(a, Pair.new(Real.0, Real.0)).first = a.first - Real.0
    a.first - Real.0 = a.first
    r2_sub(a, Pair.new(Real.0, Real.0)).second =
        a.second - Pair.new(Real.0, Real.0).second
    Pair.new(Real.0, Real.0).second = Real.0
    r2_sub(a, Pair.new(Real.0, Real.0)).second = a.second - Real.0
    a.second - Real.0 = a.second
    pair_ext(r2_sub(a, Pair.new(Real.0, Real.0)), a)
}

/// Subtracting the origin from a point negates it.
theorem r2_sub_zero_left(a: Pair[Real, Real]) {
    r2_sub(Pair.new(Real.0, Real.0), a) = r2_neg(a)
} by {
    r2_sub(Pair.new(Real.0, Real.0), a).first = Pair.new(Real.0, Real.0).first - a.first
    Pair.new(Real.0, Real.0).first = Real.0
    r2_sub(Pair.new(Real.0, Real.0), a).first = Real.0 - a.first
    Real.0 - a.first = -a.first
    r2_sub(Pair.new(Real.0, Real.0), a).second = Pair.new(Real.0, Real.0).second - a.second
    Pair.new(Real.0, Real.0).second = Real.0
    r2_sub(Pair.new(Real.0, Real.0), a).second = Real.0 - a.second
    Real.0 - a.second = -a.second
    r2_neg(a).first = -a.first
    r2_neg(a).second = -a.second
    pair_ext(r2_sub(Pair.new(Real.0, Real.0), a), r2_neg(a))
}

/// Subtracting two points successively equals subtracting their sum:
/// (x - y) - z = x - (y + z).
theorem r2_sub_sub_eq_sub_add(
    x: Pair[Real, Real], y: Pair[Real, Real], z: Pair[Real, Real]
) {
    r2_sub(r2_sub(x, y), z) = r2_sub(x, r2_add(y, z))
} by {
    let lhs = r2_sub(r2_sub(x, y), z)
    let rhs = r2_sub(x, r2_add(y, z))
    lhs.first = r2_sub(x, y).first - z.first
    r2_sub(x, y).first = x.first - y.first
    lhs.first = (x.first - y.first) - z.first
    rhs.first = x.first - r2_add(y, z).first
    r2_add(y, z).first = y.first + z.first
    rhs.first = x.first - (y.first + z.first)
    (x.first - y.first) - z.first = x.first - (y.first + z.first)
    lhs.first = rhs.first
    lhs.second = r2_sub(x, y).second - z.second
    r2_sub(x, y).second = x.second - y.second
    lhs.second = (x.second - y.second) - z.second
    rhs.second = x.second - r2_add(y, z).second
    r2_add(y, z).second = y.second + z.second
    rhs.second = x.second - (y.second + z.second)
    (x.second - y.second) - z.second = x.second - (y.second + z.second)
    lhs.second = rhs.second
    pair_ext(lhs, rhs)
}

/// Subtracting a sum successively: x - (y + z) = (x - y) - z.
theorem r2_sub_add_eq_sub_sub(
    x: Pair[Real, Real], y: Pair[Real, Real], z: Pair[Real, Real]
) {
    r2_sub(x, r2_add(y, z)) = r2_sub(r2_sub(x, y), z)
} by {
    r2_sub_sub_eq_sub_add(x, y, z)
    r2_sub(r2_sub(x, y), z) = r2_sub(x, r2_add(y, z))
    r2_sub(x, r2_add(y, z)) = r2_sub(r2_sub(x, y), z)
}

/// A point added to a difference that ends at it is the end point:
/// x + (y - x) = y.
theorem r2_add_sub_self_right(x: Pair[Real, Real], y: Pair[Real, Real]) {
    r2_add(x, r2_sub(y, x)) = y
} by {
    r2_add_comm(x, r2_sub(y, x))
    r2_add(x, r2_sub(y, x)) = r2_add(r2_sub(y, x), x)
    r2_sub_add_right_cancel(y, x)
    r2_add(r2_sub(y, x), x) = y
}

// ---------------------------------------------------------------------------
// The two displacement identities behind the nine-point circle.
// ---------------------------------------------------------------------------

/// The sum of the circumcenter and the candidate orthocenter collapses:
/// o + ((a+b+c) - 2o) = (a+b+c) - o.
theorem nine_o_add_sub_double(o: Pair[Real, Real], s: Pair[Real, Real]) {
    r2_add(o, r2_sub(s, r2_add(o, o))) = r2_sub(s, o)
} by {
    r2_sub_add_eq_sub_sub(s, o, o)
    r2_sub(s, r2_add(o, o)) = r2_sub(r2_sub(s, o), o)
    r2_add(o, r2_sub(s, r2_add(o, o))) = r2_add(o, r2_sub(r2_sub(s, o), o))
    r2_add(o, r2_sub(r2_sub(s, o), o)) = r2_sub(r2_add(o, r2_sub(s, o)), o)
    r2_add_sub_self_right(o, s)
    r2_add(o, r2_sub(s, o)) = s
    r2_sub(r2_add(o, r2_sub(s, o)), o) = r2_sub(s, o)
    r2_add(o, r2_sub(s, r2_add(o, o))) = r2_sub(s, o)
}

/// Removing a completed segment from a difference:
/// x - ((x + c) - o) = o - c.
theorem nine_sub_sum_sub(
    x: Pair[Real, Real], c: Pair[Real, Real], o: Pair[Real, Real]
) {
    r2_sub(x, r2_sub(r2_add(x, c), o)) = r2_sub(o, c)
} by {
    // (x + c) - o = x + (c - o)
    r2_sub_eq_add_neg(r2_add(x, c), o)
    r2_sub(r2_add(x, c), o) = r2_add(r2_add(x, c), r2_neg(o))
    r2_add_three_rotate(x, c, r2_neg(o))
    r2_add(r2_add(x, c), r2_neg(o)) = r2_add(r2_add(c, r2_neg(o)), x)
    r2_add_comm(r2_add(c, r2_neg(o)), x)
    r2_add(r2_add(c, r2_neg(o)), x) = r2_add(x, r2_add(c, r2_neg(o)))
    r2_sub_eq_add_neg(c, o)
    r2_sub(c, o) = r2_add(c, r2_neg(o))
    r2_add(x, r2_add(c, r2_neg(o))) = r2_add(x, r2_sub(c, o))
    r2_sub(r2_add(x, c), o) = r2_add(x, r2_sub(c, o))
    // x - (x + (c - o)) = (x - x) - (c - o) = 0 - (c - o) = o - c
    r2_sub(x, r2_sub(r2_add(x, c), o)) = r2_sub(x, r2_add(x, r2_sub(c, o)))
    r2_sub_add_eq_sub_sub(x, x, r2_sub(c, o))
    r2_sub(x, r2_add(x, r2_sub(c, o))) = r2_sub(r2_sub(x, x), r2_sub(c, o))
    r2_sub_self_zero(x)
    r2_sub(x, x) = Pair.new(Real.0, Real.0)
    r2_sub(r2_sub(x, x), r2_sub(c, o)) = r2_sub(Pair.new(Real.0, Real.0), r2_sub(c, o))
    r2_sub_zero_left(r2_sub(c, o))
    r2_sub(Pair.new(Real.0, Real.0), r2_sub(c, o)) = r2_neg(r2_sub(c, o))
    r2_sub_reverse_neg(o, c)
    r2_sub(o, c) = r2_neg(r2_sub(c, o))
    r2_neg(r2_sub(c, o)) = r2_sub(o, c)
    r2_sub(r2_sub(x, x), r2_sub(c, o)) = r2_sub(o, c)
    r2_sub(x, r2_sub(r2_add(x, c), o)) = r2_sub(o, c)
}

/// The doubled displacement from the nine-point center to a side midpoint
/// is the displacement from the circumcenter to the opposite vertex:
/// with `2m = a + b` and `2n = o + h`, `2 (m - n) = o - c`.
theorem nine_point_side_mid_identity(
    o: Pair[Real, Real], a: Pair[Real, Real], b: Pair[Real, Real],
    c: Pair[Real, Real], h: Pair[Real, Real],
    m: Pair[Real, Real], n: Pair[Real, Real]
) {
    r2_sub(r2_add(r2_add(a, b), c), r2_add(o, o)) = h and
    r2_add(a, b) = r2_add(m, m) and
    r2_add(o, h) = r2_add(n, n)
    implies
    r2_add(r2_sub(m, n), r2_sub(m, n)) = r2_sub(o, c)
} by {
    if r2_sub(r2_add(r2_add(a, b), c), r2_add(o, o)) = h and
        r2_add(a, b) = r2_add(m, m) and
        r2_add(o, h) = r2_add(n, n) {
        r2_sub(r2_add(r2_add(a, b), c), r2_add(o, o)) = h
        r2_add(a, b) = r2_add(m, m)
        r2_add(o, h) = r2_add(n, n)
        // 2(m - n) = 2m - 2n = (a + b) - (o + h)
        r2_double_sub_distrib(m, n)
        r2_add(r2_sub(m, n), r2_sub(m, n)) = r2_sub(r2_add(m, m), r2_add(n, n))
        r2_sub(r2_add(m, m), r2_add(n, n)) = r2_sub(r2_add(a, b), r2_add(o, h))
        r2_add(r2_sub(m, n), r2_sub(m, n)) = r2_sub(r2_add(a, b), r2_add(o, h))
        // o + h = o + ((a+b+c) - 2o) = (a+b+c) - o
        r2_add(o, h) = r2_add(o, r2_sub(r2_add(r2_add(a, b), c), r2_add(o, o)))
        nine_o_add_sub_double(o, r2_add(r2_add(a, b), c))
        r2_add(o, r2_sub(r2_add(r2_add(a, b), c), r2_add(o, o))) =
            r2_sub(r2_add(r2_add(a, b), c), o)
        r2_add(o, h) = r2_sub(r2_add(r2_add(a, b), c), o)
        // (a + b) - (o + h) = (a + b) - ((a+b+c) - o) = o - c
        r2_sub(r2_add(a, b), r2_add(o, h)) =
            r2_sub(r2_add(a, b), r2_sub(r2_add(r2_add(a, b), c), o))
        nine_sub_sum_sub(r2_add(a, b), c, o)
        r2_sub(r2_add(a, b), r2_sub(r2_add(r2_add(a, b), c), o)) = r2_sub(o, c)
        r2_sub(r2_add(a, b), r2_add(o, h)) = r2_sub(o, c)
        r2_add(r2_sub(m, n), r2_sub(m, n)) = r2_sub(o, c)
    }
}

/// The doubled displacement from the nine-point center to an Euler point
/// (the midpoint of a vertex and the orthocenter) is the displacement from
/// the vertex to the circumcenter: with `2e = a + h` and `2n = o + h`,
/// `2 (e - n) = a - o`.
theorem nine_point_euler_mid_identity(
    o: Pair[Real, Real], a: Pair[Real, Real], h: Pair[Real, Real],
    e: Pair[Real, Real], n: Pair[Real, Real]
) {
    r2_add(a, h) = r2_add(e, e) and
    r2_add(o, h) = r2_add(n, n)
    implies
    r2_add(r2_sub(e, n), r2_sub(e, n)) = r2_sub(a, o)
} by {
    if r2_add(a, h) = r2_add(e, e) and r2_add(o, h) = r2_add(n, n) {
        r2_add(a, h) = r2_add(e, e)
        r2_add(o, h) = r2_add(n, n)
        // 2(e - n) = 2e - 2n = (a + h) - (o + h) = (a - o) + (h - h)
        r2_double_sub_distrib(e, n)
        r2_add(r2_sub(e, n), r2_sub(e, n)) = r2_sub(r2_add(e, e), r2_add(n, n))
        r2_sub(r2_add(e, e), r2_add(n, n)) = r2_sub(r2_add(a, h), r2_add(o, h))
        r2_add(r2_sub(e, n), r2_sub(e, n)) = r2_sub(r2_add(a, h), r2_add(o, h))
        r2_sub_add_pair_distrib(a, h, o, h)
        r2_sub(r2_add(a, h), r2_add(o, h)) = r2_add(r2_sub(a, o), r2_sub(h, h))
        r2_sub_self_zero(h)
        r2_sub(h, h) = Pair.new(Real.0, Real.0)
        r2_add(r2_sub(a, o), r2_sub(h, h)) = r2_add(r2_sub(a, o), Pair.new(Real.0, Real.0))
        r2_add_zero_right(r2_sub(a, o))
        r2_add(r2_sub(a, o), Pair.new(Real.0, Real.0)) = r2_sub(a, o)
        r2_sub(r2_add(a, h), r2_add(o, h)) = r2_sub(a, o)
        r2_add(r2_sub(e, n), r2_sub(e, n)) = r2_sub(a, o)
    }
}

/// The side midpoints of a triangle lie on the nine-point circle.
///
/// With `m_ab`, `m_bc`, `m_ca` the midpoints of the sides, the nine-point
/// center `n` (midpoint of the circumcenter `o` and the orthocenter `h`)
/// satisfies `4 |m - n|^2 = radius_sq` for each side midpoint, i.e. all
/// three side midpoints lie on the circle with center `n` and squared
/// radius `radius_sq / 4`.
theorem theorems1000_nine_point_side_midpoints(
    o: Pair[Real, Real], radius_sq: Real,
    a: Pair[Real, Real], b: Pair[Real, Real], c: Pair[Real, Real],
    h: Pair[Real, Real], n: Pair[Real, Real],
    m_ab: Pair[Real, Real], m_bc: Pair[Real, Real], m_ca: Pair[Real, Real]
) {
    r2_norm_sq(r2_sub(a, o)) = radius_sq and
    r2_norm_sq(r2_sub(b, o)) = radius_sq and
    r2_norm_sq(r2_sub(c, o)) = radius_sq and
    r2_sub(r2_add(r2_add(a, b), c), r2_add(o, o)) = h and
    r2_add(o, h) = r2_add(n, n) and
    r2_add(a, b) = r2_add(m_ab, m_ab) and
    r2_add(b, c) = r2_add(m_bc, m_bc) and
    r2_add(c, a) = r2_add(m_ca, m_ca)
    implies
    (Real.1 + Real.1) * ((Real.1 + Real.1) * r2_norm_sq(r2_sub(m_ab, n))) = radius_sq and
    (Real.1 + Real.1) * ((Real.1 + Real.1) * r2_norm_sq(r2_sub(m_bc, n))) = radius_sq and
    (Real.1 + Real.1) * ((Real.1 + Real.1) * r2_norm_sq(r2_sub(m_ca, n))) = radius_sq
} by {
    if r2_norm_sq(r2_sub(a, o)) = radius_sq and
        r2_norm_sq(r2_sub(b, o)) = radius_sq and
        r2_norm_sq(r2_sub(c, o)) = radius_sq and
        r2_sub(r2_add(r2_add(a, b), c), r2_add(o, o)) = h and
        r2_add(o, h) = r2_add(n, n) and
        r2_add(a, b) = r2_add(m_ab, m_ab) and
        r2_add(b, c) = r2_add(m_bc, m_bc) and
        r2_add(c, a) = r2_add(m_ca, m_ca) {
        r2_norm_sq(r2_sub(a, o)) = radius_sq
        r2_norm_sq(r2_sub(b, o)) = radius_sq
        r2_norm_sq(r2_sub(c, o)) = radius_sq
        r2_sub(r2_add(r2_add(a, b), c), r2_add(o, o)) = h
        r2_add(o, h) = r2_add(n, n)
        r2_add(a, b) = r2_add(m_ab, m_ab)
        r2_add(b, c) = r2_add(m_bc, m_bc)
        r2_add(c, a) = r2_add(m_ca, m_ca)
        // the midpoint of ab: 2(m_ab - n) = o - c
        nine_point_side_mid_identity(o, a, b, c, h, m_ab, n)
        r2_add(r2_sub(m_ab, n), r2_sub(m_ab, n)) = r2_sub(o, c)
        // |o - c|^2 = |c - o|^2 = radius_sq
        r2_sub_reverse_neg(c, o)
        r2_sub(c, o) = r2_neg(r2_sub(o, c))
        r2_norm_sq_neg(r2_sub(o, c))
        r2_norm_sq(r2_neg(r2_sub(o, c))) = r2_norm_sq(r2_sub(o, c))
        r2_norm_sq(r2_sub(c, o)) = r2_norm_sq(r2_sub(o, c))
        r2_norm_sq(r2_sub(o, c)) = radius_sq
        // 4 |m_ab - n|^2 = |2 (m_ab - n)|^2 = radius_sq
        r2_norm_sq_double(r2_sub(m_ab, n))
        r2_norm_sq(r2_add(r2_sub(m_ab, n), r2_sub(m_ab, n))) =
            (Real.1 + Real.1) * ((Real.1 + Real.1) * r2_norm_sq(r2_sub(m_ab, n)))
        r2_norm_sq(r2_add(r2_sub(m_ab, n), r2_sub(m_ab, n))) = r2_norm_sq(r2_sub(o, c))
        (Real.1 + Real.1) * ((Real.1 + Real.1) * r2_norm_sq(r2_sub(m_ab, n))) = radius_sq
        // the midpoint of bc: rotate (a, b, c) to (b, c, a)
        r2_add_three_rotate(a, b, c)
        r2_add(r2_add(a, b), c) = r2_add(r2_add(b, c), a)
        r2_sub(r2_add(r2_add(b, c), a), r2_add(o, o)) = h
        nine_point_side_mid_identity(o, b, c, a, h, m_bc, n)
        r2_add(r2_sub(m_bc, n), r2_sub(m_bc, n)) = r2_sub(o, a)
        r2_sub_reverse_neg(a, o)
        r2_sub(a, o) = r2_neg(r2_sub(o, a))
        r2_norm_sq_neg(r2_sub(o, a))
        r2_norm_sq(r2_neg(r2_sub(o, a))) = r2_norm_sq(r2_sub(o, a))
        r2_norm_sq(r2_sub(a, o)) = r2_norm_sq(r2_sub(o, a))
        r2_norm_sq(r2_sub(o, a)) = radius_sq
        r2_norm_sq_double(r2_sub(m_bc, n))
        r2_norm_sq(r2_add(r2_sub(m_bc, n), r2_sub(m_bc, n))) =
            (Real.1 + Real.1) * ((Real.1 + Real.1) * r2_norm_sq(r2_sub(m_bc, n)))
        r2_norm_sq(r2_add(r2_sub(m_bc, n), r2_sub(m_bc, n))) = r2_norm_sq(r2_sub(o, a))
        (Real.1 + Real.1) * ((Real.1 + Real.1) * r2_norm_sq(r2_sub(m_bc, n))) = radius_sq
        // the midpoint of ca: rotate (a, b, c) to (c, a, b)
        r2_add_three_rotate(c, a, b)
        r2_add(r2_add(c, a), b) = r2_add(r2_add(a, b), c)
        r2_sub(r2_add(r2_add(c, a), b), r2_add(o, o)) = h
        nine_point_side_mid_identity(o, c, a, b, h, m_ca, n)
        r2_add(r2_sub(m_ca, n), r2_sub(m_ca, n)) = r2_sub(o, b)
        r2_sub_reverse_neg(b, o)
        r2_sub(b, o) = r2_neg(r2_sub(o, b))
        r2_norm_sq_neg(r2_sub(o, b))
        r2_norm_sq(r2_neg(r2_sub(o, b))) = r2_norm_sq(r2_sub(o, b))
        r2_norm_sq(r2_sub(b, o)) = r2_norm_sq(r2_sub(o, b))
        r2_norm_sq(r2_sub(o, b)) = radius_sq
        r2_norm_sq_double(r2_sub(m_ca, n))
        r2_norm_sq(r2_add(r2_sub(m_ca, n), r2_sub(m_ca, n))) =
            (Real.1 + Real.1) * ((Real.1 + Real.1) * r2_norm_sq(r2_sub(m_ca, n)))
        r2_norm_sq(r2_add(r2_sub(m_ca, n), r2_sub(m_ca, n))) = r2_norm_sq(r2_sub(o, b))
        (Real.1 + Real.1) * ((Real.1 + Real.1) * r2_norm_sq(r2_sub(m_ca, n))) = radius_sq
        (Real.1 + Real.1) * ((Real.1 + Real.1) * r2_norm_sq(r2_sub(m_ab, n))) = radius_sq and
            (Real.1 + Real.1) * ((Real.1 + Real.1) * r2_norm_sq(r2_sub(m_bc, n))) = radius_sq and
            (Real.1 + Real.1) * ((Real.1 + Real.1) * r2_norm_sq(r2_sub(m_ca, n))) = radius_sq
    }
}

/// The Euler points of a triangle lie on the nine-point circle.
///
/// With `e_a`, `e_b`, `e_c` the midpoints of the segments from the vertices
/// to the orthocenter, the nine-point center `n` satisfies
/// `4 |e - n|^2 = radius_sq` for each Euler point.
theorem theorems1000_nine_point_euler_midpoints(
    o: Pair[Real, Real], radius_sq: Real,
    a: Pair[Real, Real], b: Pair[Real, Real], c: Pair[Real, Real],
    h: Pair[Real, Real], n: Pair[Real, Real],
    e_a: Pair[Real, Real], e_b: Pair[Real, Real], e_c: Pair[Real, Real]
) {
    r2_norm_sq(r2_sub(a, o)) = radius_sq and
    r2_norm_sq(r2_sub(b, o)) = radius_sq and
    r2_norm_sq(r2_sub(c, o)) = radius_sq and
    r2_sub(r2_add(r2_add(a, b), c), r2_add(o, o)) = h and
    r2_add(o, h) = r2_add(n, n) and
    r2_add(a, h) = r2_add(e_a, e_a) and
    r2_add(b, h) = r2_add(e_b, e_b) and
    r2_add(c, h) = r2_add(e_c, e_c)
    implies
    (Real.1 + Real.1) * ((Real.1 + Real.1) * r2_norm_sq(r2_sub(e_a, n))) = radius_sq and
    (Real.1 + Real.1) * ((Real.1 + Real.1) * r2_norm_sq(r2_sub(e_b, n))) = radius_sq and
    (Real.1 + Real.1) * ((Real.1 + Real.1) * r2_norm_sq(r2_sub(e_c, n))) = radius_sq
} by {
    if r2_norm_sq(r2_sub(a, o)) = radius_sq and
        r2_norm_sq(r2_sub(b, o)) = radius_sq and
        r2_norm_sq(r2_sub(c, o)) = radius_sq and
        r2_sub(r2_add(r2_add(a, b), c), r2_add(o, o)) = h and
        r2_add(o, h) = r2_add(n, n) and
        r2_add(a, h) = r2_add(e_a, e_a) and
        r2_add(b, h) = r2_add(e_b, e_b) and
        r2_add(c, h) = r2_add(e_c, e_c) {
        r2_norm_sq(r2_sub(a, o)) = radius_sq
        r2_norm_sq(r2_sub(b, o)) = radius_sq
        r2_norm_sq(r2_sub(c, o)) = radius_sq
        r2_sub(r2_add(r2_add(a, b), c), r2_add(o, o)) = h
        r2_add(o, h) = r2_add(n, n)
        r2_add(a, h) = r2_add(e_a, e_a)
        r2_add(b, h) = r2_add(e_b, e_b)
        r2_add(c, h) = r2_add(e_c, e_c)
        // the Euler point over a: 2(e_a - n) = a - o
        nine_point_euler_mid_identity(o, a, h, e_a, n)
        r2_add(r2_sub(e_a, n), r2_sub(e_a, n)) = r2_sub(a, o)
        r2_norm_sq_double(r2_sub(e_a, n))
        r2_norm_sq(r2_add(r2_sub(e_a, n), r2_sub(e_a, n))) =
            (Real.1 + Real.1) * ((Real.1 + Real.1) * r2_norm_sq(r2_sub(e_a, n)))
        r2_norm_sq(r2_add(r2_sub(e_a, n), r2_sub(e_a, n))) = r2_norm_sq(r2_sub(a, o))
        (Real.1 + Real.1) * ((Real.1 + Real.1) * r2_norm_sq(r2_sub(e_a, n))) = radius_sq
        // the Euler point over b: 2(e_b - n) = b - o
        nine_point_euler_mid_identity(o, b, h, e_b, n)
        r2_add(r2_sub(e_b, n), r2_sub(e_b, n)) = r2_sub(b, o)
        r2_norm_sq_double(r2_sub(e_b, n))
        r2_norm_sq(r2_add(r2_sub(e_b, n), r2_sub(e_b, n))) =
            (Real.1 + Real.1) * ((Real.1 + Real.1) * r2_norm_sq(r2_sub(e_b, n)))
        r2_norm_sq(r2_add(r2_sub(e_b, n), r2_sub(e_b, n))) = r2_norm_sq(r2_sub(b, o))
        (Real.1 + Real.1) * ((Real.1 + Real.1) * r2_norm_sq(r2_sub(e_b, n))) = radius_sq
        // the Euler point over c: 2(e_c - n) = c - o
        nine_point_euler_mid_identity(o, c, h, e_c, n)
        r2_add(r2_sub(e_c, n), r2_sub(e_c, n)) = r2_sub(c, o)
        r2_norm_sq_double(r2_sub(e_c, n))
        r2_norm_sq(r2_add(r2_sub(e_c, n), r2_sub(e_c, n))) =
            (Real.1 + Real.1) * ((Real.1 + Real.1) * r2_norm_sq(r2_sub(e_c, n)))
        r2_norm_sq(r2_add(r2_sub(e_c, n), r2_sub(e_c, n))) = r2_norm_sq(r2_sub(c, o))
        (Real.1 + Real.1) * ((Real.1 + Real.1) * r2_norm_sq(r2_sub(e_c, n))) = radius_sq
        (Real.1 + Real.1) * ((Real.1 + Real.1) * r2_norm_sq(r2_sub(e_a, n))) = radius_sq and
            (Real.1 + Real.1) * ((Real.1 + Real.1) * r2_norm_sq(r2_sub(e_b, n))) = radius_sq and
            (Real.1 + Real.1) * ((Real.1 + Real.1) * r2_norm_sq(r2_sub(e_c, n))) = radius_sq
    }
}

// The full nine-point circle theorem: the three families of nine points all
// lie on the circle with center `n` and squared radius `radius_sq / 4`.
// The side midpoints and the Euler points are proved above; the altitude
// feet remain.  With `f_a` the foot of the altitude from `a` onto `bc`,
// written as the affine combination `f_a = b + s (c - b)` with the parameter
// `s` fixed by `(a - f_a) . (c - b) = 0`, the computation of `|f_a - n|^2`
// expands, as in the geometric-mean theorem, into a rational expression in
// `s` and the side lengths, and the same norm-square algebra gives
// `4 |f_a - n|^2 = radius_sq`; the cyclic rotations cover the other two
// feet.  The statement is recorded here with the altitude-foot hypotheses.
//
// theorem theorems1000_nine_point_circle(
//     o: Pair[Real, Real], radius_sq: Real,
//     a: Pair[Real, Real], b: Pair[Real, Real], c: Pair[Real, Real],
//     h: Pair[Real, Real], n: Pair[Real, Real],
//     f_a: Pair[Real, Real], f_b: Pair[Real, Real], f_c: Pair[Real, Real]
// ) {
//     r2_norm_sq(r2_sub(a, o)) = radius_sq and
//     r2_norm_sq(r2_sub(b, o)) = radius_sq and
//     r2_norm_sq(r2_sub(c, o)) = radius_sq and
//     r2_sub(r2_add(r2_add(a, b), c), r2_add(o, o)) = h and
//     r2_add(o, h) = r2_add(n, n) and
//     r2_dot(r2_sub(a, f_a), r2_sub(c, b)) = Real.0 and
//     r2_dot(r2_sub(b, f_b), r2_sub(a, c)) = Real.0 and
//     r2_dot(r2_sub(c, f_c), r2_sub(b, a)) = Real.0
//     implies
//     (Real.1 + Real.1) * ((Real.1 + Real.1) * r2_norm_sq(r2_sub(f_a, n))) = radius_sq and
//     (Real.1 + Real.1) * ((Real.1 + Real.1) * r2_norm_sq(r2_sub(f_b, n))) = radius_sq and
//     (Real.1 + Real.1) * ((Real.1 + Real.1) * r2_norm_sq(r2_sub(f_c, n))) = radius_sq
// }
