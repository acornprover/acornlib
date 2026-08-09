from nat import Nat
from real import Real, converges, limit, converges_to, is_increasing, is_upper_bound,
    monotone_convergence_principle, increasing_convergent_bounded_by_limit,
    converges_imp_converges_to, is_set_upper_bound, is_set_supremum, eventual_ub,
    ub_imp_limit_lte
from data.basic.set import Set, set_image, set_image_contains_witness,
    maps_into_set_image, universal_set_contains_eq

numerals Real

// The monotone convergence theorem (the monotone convergence principle): a
// nondecreasing sequence of real numbers that is bounded above converges,
// and its limit is the supremum of its values.  Equivalently, every bounded
// monotone sequence of reals converges (a nonincreasing sequence bounded
// below converges by the same argument applied to its negation).  The
// theorem is the analytic form of the completeness of the real numbers: it
// is equivalent to the existence of suprema, and it is the engine behind
// the convergence of the partial sums of nonnegative series, the
// convergence of the iterates of a contraction, and the construction of
// the exponential and trigonometric functions by power series.
//
// The first statement below is the convergence half: an increasing sequence
// `a` with an upper bound `b` (every term satisfies `a(n) <= b`) converges
// to its limit `limit(a)`.  The second statement is the completeness half:
// the limit is the least upper bound (supremum) of the set of values
// `{ a(n) : n in Nat }`, encoded as the set image `seq_image(a)`.
//
// Proof of the second statement: by the first statement the sequence
// converges, and `increasing_convergent_bounded_by_limit` gives that every
// term is below the limit, so the limit is an upper bound of the value set.
// Conversely, if `ub` bounds every value, then `ub` is an upper bound of
// the sequence, and `ub_imp_limit_lte` gives `limit(a) <= ub`; hence the
// limit is the least upper bound.

/// The value set of a sequence: the set image of the naturals under `a`.
define seq_image(a: Nat -> Real) -> Set[Real] {
    set_image(Set[Nat].universal_set, a)
}

/// The monotone convergence theorem: a nondecreasing sequence bounded above
/// converges.
theorem theorems1000_monotone_convergence(a: Nat -> Real, b: Real) {
    is_increasing(a) and is_upper_bound(a, b) implies converges(a)
} by {
    if is_increasing(a) and is_upper_bound(a, b) {
        monotone_convergence_principle(a, b)
        converges(a)
    }
}

/// The limit of an increasing bounded sequence is the supremum of its
/// values.
theorem theorems1000_monotone_convergence_limit_supremum(a: Nat -> Real, b: Real) {
    is_increasing(a) and is_upper_bound(a, b)
    implies is_set_supremum(seq_image(a), limit(a))
} by {
    if is_increasing(a) and is_upper_bound(a, b) {
        // The sequence converges and every term lies below the limit.
        theorems1000_monotone_convergence(a, b)
        converges(a)
        increasing_convergent_bounded_by_limit(a)
        is_upper_bound(a, limit(a))
        // The limit is an upper bound of the value set.
        forall(x: Real) {
            if seq_image(a).contains(x) {
                set_image_contains_witness(Set[Nat].universal_set, a, x)
                exists(n: Nat) {
                    Set[Nat].universal_set.contains(n) and x = a(n)
                }
                let n: Nat satisfy {
                    Set[Nat].universal_set.contains(n) and x = a(n)
                }
                a(n) <= limit(a)
                x = a(n)
                x <= limit(a)
            }
        }
        is_set_upper_bound(seq_image(a), limit(a))
        // Every upper bound of the value set bounds the sequence from some
        // index on, so the limit lies below it.
        forall(ub: Real) {
            if is_set_upper_bound(seq_image(a), ub) {
                forall(n: Nat) {
                    universal_set_contains_eq[Nat](n)
                    Set[Nat].universal_set.contains(n)
                    maps_into_set_image(Set[Nat].universal_set, a, n)
                    seq_image(a).contains(a(n))
                    is_set_upper_bound(seq_image(a), ub) = forall(x: Real) {
                        seq_image(a).contains(x) implies x <= ub
                    }
                    seq_image(a).contains(a(n)) implies a(n) <= ub
                    a(n) <= ub
                }
                is_upper_bound(a, ub)
                // An upper bound of the sequence is an eventual upper bound.
                (eventual_ub(a, ub) = exists(n0: Nat) {
                    forall(i: Nat) {
                        n0 <= i implies a(i) <= ub
                    }
                })
                exists(n0: Nat) {
                    forall(i: Nat) {
                        n0 <= i implies a(i) <= ub
                    }
                }
                eventual_ub(a, ub)
                ub_imp_limit_lte(a, ub)
                limit(a) <= ub
            }
        }
        is_set_supremum(seq_image(a), limit(a))
    }
}
