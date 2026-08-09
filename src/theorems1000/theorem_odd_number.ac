from list import partial, partial_zero, partial_split_last
from nat import Nat
numerals Nat

// The odd number theorem: the sum of the first `n` odd numbers
//     1 + 3 + 5 + ... + (2n - 1) = n^2.
// (An entry of the "1000+ theorems" list.)

/// The k-th odd number `2k + 1`, in zero-based indexing (k = 0 gives 1).
define odd_number(k: Nat) -> Nat {
    k * Nat.2 + Nat.1
}

/// Step of the induction: `n^2 + (2n + 1) = (n + 1)^2`.
theorem odd_number_step(n: Nat) {
    n * n + (n * Nat.2 + Nat.1) = n.suc * n.suc
} by {
    n.suc * n.suc = (n + Nat.1) * (n + Nat.1)
    (n + Nat.1) * (n + Nat.1) = (n + Nat.1) * n + (n + Nat.1) * Nat.1
    (n + Nat.1) * n = n * n + n
    (n + Nat.1) * Nat.1 = n + Nat.1
    (n + Nat.1) * (n + Nat.1) = n * n + n + (n + Nat.1)
    n * n + n + (n + Nat.1) = n * n + (n + n) + Nat.1
    n + n = n * Nat.2
    n * n + (n * Nat.2 + Nat.1) = n * n + (n + n) + Nat.1
    n.suc * n.suc = n * n + (n * Nat.2 + Nat.1)
}

/// The odd number theorem: the sum of the first `n` odd numbers is `n^2`.
theorem theorems1000_odd_number(n: Nat) {
    partial[Nat](odd_number, n) = n * n
} by {
    define p(x: Nat) -> Bool {
        partial[Nat](odd_number, x) = x * x
    }

    partial_zero[Nat](odd_number)
    partial[Nat](odd_number, Nat.0) = Nat.0
    Nat.0 * Nat.0 = Nat.0
    p(Nat.0)

    forall(x: Nat) {
        if p(x) {
            partial[Nat](odd_number, x) = x * x
            partial_split_last[Nat](odd_number, x)
            partial[Nat](odd_number, x.suc) = partial[Nat](odd_number, x) + odd_number(x)
            partial[Nat](odd_number, x.suc) = x * x + odd_number(x)
            odd_number(x) = x * Nat.2 + Nat.1
            partial[Nat](odd_number, x.suc) = x * x + (x * Nat.2 + Nat.1)
            odd_number_step(x)
            x * x + (x * Nat.2 + Nat.1) = x.suc * x.suc
            partial[Nat](odd_number, x.suc) = x.suc * x.suc
            p(x.suc)
        }
    }
    p(n)
}
