from real import Real, continuous, intermediate_value_closed_interval
from order_set import closed_interval_set, closed_interval_set_lower_le,
    closed_interval_set_le_upper

numerals Real

// The intermediate value theorem (Bernard Bolzano, 1817; Augustin-Louis
// Cauchy, 1821): a continuous function on an interval takes every value
// between its values at the endpoints.  If `f` is continuous and
// `f(a) <= t <= f(b)` with `a <= b`, then some point `c` of `[a, b]`
// satisfies `f(c) = t`.  The theorem is the fundamental "no jumps" property
// of continuity: the image of an interval under a continuous function is an
// interval.  It underlies the existence of roots (via sign changes, see
// `theorem_bolzano.ac` in this directory for the zero-target form), the
// existence of `n`-th roots, and the proof that every odd-degree real
// polynomial has a real root.  The special case with target `t = 0` is
// Bolzano's theorem; conversely, the intermediate value theorem follows
// from Bolzano's theorem by applying it to `f - t`.  Both Bolzano's theorem
// and the intermediate value theorem are entries of the "1000+ theorems"
// list (in that order), and this file records the general form.
//
// The formal statement uses the library's set notation: the closed interval
// is `closed_interval_set(a, b)`, whose elements are the reals `x` with
// `a <= x <= b`, and the conclusion is the existence of a point `c` of the
// interval with `f(c) = t`.
//
// Proof: the library's `intermediate_value_closed_interval` in
// `real/interface.ac` proves exactly this statement (its proof bisects the
// interval, using the completeness of the reals); we record the theorem
// here as an entry of the "1000+ theorems" list and re-derive it from that
// result, unpacking the interval membership into the endpoint
// inequalities.

/// The intermediate value theorem: a continuous function takes every value
/// between its endpoint values on an interval.
theorem theorems1000_intermediate_value(f: Real -> Real, a: Real, b: Real, t: Real) {
    continuous(f) and a <= b and f(a) <= t and t <= f(b)
    implies exists(c: Real) { a <= c and c <= b and f(c) = t }
} by {
    if continuous(f) and a <= b and f(a) <= t and t <= f(b) {
        intermediate_value_closed_interval(f, a, b, t)
        exists(point: Real) {
            closed_interval_set(a, b).contains(point) and f(point) = t
        }
        let point: Real satisfy {
            closed_interval_set(a, b).contains(point) and f(point) = t
        }
        closed_interval_set_lower_le(a, b, point)
        a <= point
        closed_interval_set_le_upper(a, b, point)
        point <= b
        exists(c: Real) { a <= c and c <= b and f(c) = t }
    }
}
