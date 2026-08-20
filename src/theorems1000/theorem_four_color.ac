from data.fin.fin import Fin
from graph import SimpleGraph, is_planar, simple_graph_coloring

// The four color theorem (Francis Guthrie, 1852; proved by Kenneth Appel
// and Wolfgang Haken, 1976): every planar map can be colored with four
// colors so that no two adjacent regions share a color.  Equivalently,
// every planar graph is 4-colorable: its vertices can be colored with four
// colors so that adjacent vertices get different colors.  The theorem is
// the first major result proved with substantial computer assistance; the
// proof reduces to 1936 unavoidable configurations, each checked by
// program, and was later simplified by Robertson, Sanders, Seymour and
// Thomas (1997).  Five colors have been known since the 1890s (Heawood,
// via Kempe chains); the step from five to four is famously delicate.
//
// The formal statement uses the library's graph machinery: `SimpleGraph[V]`
// is a simple graph on the vertex type `V`, `is_planar(g)` is the
// library's planarity predicate, and `simple_graph_coloring(g, color)` says
// that the map `color: V -> Fin[Nat.4]` is a proper 4-coloring (adjacent
// vertices receive distinct colors).
//
// Caveat: the library's `is_planar` (in `graph/planar_euler.ac`) is a
// documented placeholder that is currently the constant `false` until a
// genuine topological or combinatorial embedding definition is formalized,
// so the statement below is vacuous at present; it records the theorem for
// when planarity is available.  The classical proof is a huge case
// analysis (reducibility of configurations, discharging) and is not
// expected to be formalized soon.

// theorem theorems1000_four_color[V](g: SimpleGraph[V]) {
//     is_planar(g) implies
//         exists(color: V -> Fin[Nat.4]) {
//             simple_graph_coloring(g, color)
//         }
// }
