from nat import Nat

numerals Nat

// Sophie Germain's theorem: if `p >= 3` is prime and `2 p + 1` is also
// prime (a "Sophie Germain prime" pair), then the equation
//     x^p + y^p = z^p
// has no solution in positive integers `x`, `y`, `z` none of which is
// divisible by `p`.  This is the "first case" of Fermat's last theorem for
// the exponent `p` (see theorem_fermat_last.ac in this directory).  Marie
// Sophie Germain proved the theorem in 1823, and used it to show the first
// case of Fermat's last theorem for every prime below 100.
//
// The algebraic heart of the proof is Sophie Germain's identity
//     a^4 + 4 b^4 = (a^2 + 2 a b + 2 b^2) (a^2 - 2 a b + 2 b^2),
// proved for the reals in theorem_sophie_germain.ac in this directory:
// factorising `x^p + y^p` (for odd `p`, `x^p + y^p` is divisible by
// `x + y`, and the cofactor is a sum of `p` alternating terms), the
// identity exhibits a sum of two fourth powers inside the cofactor, which
// must itself be a `p`-th power; comparing the two factorisations of the
// same number forces `p | x y z` by descent on the size of the solution.
//
// The statement is recorded in the library's natural-number arithmetic
// (`p.is_prime`, `2 p + 1` written as `Nat.2 * p + Nat.1`, and
// `p.divides(x)` for divisibility).
//
// Proof sketch (following Germain 1823): suppose `x^p + y^p = z^p` with
// `p` and `2 p + 1` prime and `p` dividing none of `x`, `y`, `z`.  Since
// `2 p + 1` is prime, the multiplicative group modulo `2 p + 1` has order
// `2 p`, so every nonzero residue has order dividing `2 p`; in particular
// no nonzero residue is a `p`-th power unless it is `1` (a `p`-th power
// has order dividing `2`, i.e. is `+-1`, and `-1` is not a `p`-th power
// because `p` is odd).  Hence `x^p, y^p, z^p` are all congruent to `+-1`
// modulo `2 p + 1`, and `x^p + y^p = z^p` forces an impossible equation of
// signs.  This congruence argument avoids the descent entirely; the
// descent version (needed for primes without the `2 p + 1` property)
// proceeds through the Sophie Germain identity above.
//
// theorem theorems1000_sophie_germain_prime {
//     forall(p: Nat, x: Nat, y: Nat, z: Nat) {
//         Nat.2 < p and p.is_prime and (Nat.2 * p + Nat.1).is_prime and
//         Nat.1 <= x and Nat.1 <= y and Nat.1 <= z and
//         not p.divides(x) and not p.divides(y) and not p.divides(z)
//             implies not (x.pow(p) + y.pow(p) = z.pow(p))
//     }
// }
