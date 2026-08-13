from real import Real, has_derivative_at, darboux_theorem

numerals Real

// Darboux's theorem (Gaston Darboux, 1875): every derivative has the
// intermediate value property.  If `f` is differentiable on an interval
// containing `a < b` and the derivative `f'` takes values `f'(a)` and
// `f'(b)` of opposite sides of a value `t`, then `f'(c) = t` for some
// `c` strictly between `a` and `b`.  The theorem is the reason derivatives
// cannot have jump discontinuities (a function with a jump, such as the
// step function, is not a derivative), and it is the classical companion
// of the intermediate value theorem for continuous functions.  Unlike the
// latter, it needs no continuity assumption on `f'`: differentiability of
// `f` alone forces the intermediate value property.
//
// The formal statement below uses the library's derivative vocabulary: the
// pointwise derivative predicate `has_derivative_at(f, x, d)` says that
// `d` is the derivative of `f` at `x` (see `real/derivative_basic.ac`),
// and `f'` is represented by the function `df` with
// `has_derivative_at(f, x, df(x))` for every `x`.  The derivative values
// `df(a)` and `df(b)` are the endpoint values.
//
// Proof: the library proves the theorem as `darboux_theorem` in
// `real/darboux.ac`, by the classical argument through the auxiliary
// function `g(x) = f(x) - t x`: `g'(a) < 0 < g'(b)`, so the minimum of
// `g` on `[a, b]` (which exists by the extreme value theorem) is attained
// at an interior point `c` (a negative derivative at `a` forces `g` below
// `g(a)` to the right of `a`, and a positive derivative at `b` forces `g`
// below `g(b)` to the left of `b`); Fermat's theorem on interior extrema
// then gives `g'(c) = 0`, i.e. `df(c) = t`.  The statement below is the
// same statement, recorded with the classical hypotheses in the order of
// the historical theorem.

/// Darboux's theorem: derivatives have the intermediate value property.
theorem theorems1000_darboux(f: Real -> Real, df: Real -> Real, a: Real, b: Real, t: Real) {
    a < b and
    (forall(x: Real) { has_derivative_at(f, x, df(x)) }) and
    df(a) < t and t < df(b)
    implies exists(c: Real) {
        a < c and c < b and df(c) = t
    }
} by {
    if a < b and (forall(x: Real) { has_derivative_at(f, x, df(x)) }) and
       df(a) < t and t < df(b) {
        darboux_theorem(f, df, a, b, t)
        exists(c: Real) {
            a < c and c < b and df(c) = t
        }
    }
}
