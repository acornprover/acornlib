from pair import Pair
from real import Real, mul_left_cancel
from top100 import r2_add, r2_neg, r2_sub, r2_norm_sq
from theorems1000.r2_helpers import r2_smul, r2_cross,
    r2_smul_one, r2_smul_neg, r2_smul_add_distrib, r2_smul_sub_distrib,
    r2_smul_sub_self, r2_sub_eq_add_neg, r2_sub_add_pair_distrib,
    r2_add_zero_right, r2_add_comm, r2_add_assoc, r2_add_neg_right,
    r2_sub_reverse_neg, r2_norm_sq_neg, r2_norm_sq_smul,
    r2_cross_add_left, r2_cross_add_right, r2_cross_smul_left, r2_cross_smul_right,
    r2_cross_swap, r2_cross_self_zero, r2_pair_ext

// The angle bisector theorem: the internal bisector of an angle of a
// triangle divides the opposite side in the ratio of the two adjacent sides.
// For the angle at `a` of the triangle `abc`, the internal bisector meets
// `bc` at a point `d` satisfying `BD/DC = AB/AC`.
//
// The internal bisector direction is the sum of the two unit vectors along
// `ab` and `ac`; with the length witnesses `u^2 = |b-a|^2` and
// `v^2 = |c-a|^2`, that direction is `v (b-a) + u (c-a)` (the unit vectors
// scaled by `u v`).  Writing `d = (1 - t) b + t c` on the segment `bc`, the
// condition that `ad` has the bisector direction is
//     cross(d - a, v (b - a) + u (c - a)) = 0,
// and the conclusion in squared-ratio form is
//     |b-d|^2 * v^2 = |d-c|^2 * u^2,
// i.e. `(BD/DC)^2 = (AB/AC)^2`.
//
// Proof: write `U = b - a` and `V = c - a`.  From `d = (1 - t) b + t c`,
// `d - a = (1 - t) U + t V`.  Expanding the cross product with the
// bilinearity lemmas gives
//     cross(d - a, v U + u V) = [(1 - t) u - t v] * cross(U, V),
// and since `cross(U, V) != 0` (a non-degenerate triangle), the zero-product
// law gives `(1 - t) u = t v`.  Finally
//     |b - d|^2 = t^2 |c - b|^2 and |d - c|^2 = (1 - t)^2 |c - b|^2,
// so `|b-d|^2 v^2 = t^2 |c-b|^2 v^2 = (1 - t)^2 |c-b|^2 u^2 = |d-c|^2 u^2`
// by squaring `(1 - t) u = t v`.

// ---------------------------------------------------------------------------
// Scalar-multiplication helpers.
// ---------------------------------------------------------------------------

/// Subtracting the origin leaves a point unchanged.
theorem angle_bisector_sub_zero_right(a: Pair[Real, Real]) {
    r2_sub(a, Pair.new(Real.0, Real.0)) = a
} by {
    let lhs = r2_sub(a, Pair.new(Real.0, Real.0))
    lhs.first = a.first - Pair.new(Real.0, Real.0).first
    Pair.new(Real.0, Real.0).first = Real.0
    lhs.first = a.first - Real.0
    a.first - Real.0 = a.first
    lhs.second = a.second - Pair.new(Real.0, Real.0).second
    Pair.new(Real.0, Real.0).second = Real.0
    lhs.second = a.second - Real.0
    a.second - Real.0 = a.second
    r2_pair_ext(lhs, a)
}

/// Scalar multiplication distributes over scalar addition:
/// `k1 a + k2 a = (k1 + k2) a`.
theorem angle_bisector_smul_add_scalar(k1: Real, k2: Real, a: Pair[Real, Real]) {
    r2_add(r2_smul(k1, a), r2_smul(k2, a)) = r2_smul(k1 + k2, a)
} by {
    let s1 = r2_smul(k1, a)
    let s2 = r2_smul(k2, a)
    let rhs = r2_smul(k1 + k2, a)
    r2_pair_ext(r2_add(s1, s2), rhs)
    r2_add(s1, s2).first = s1.first + s2.first
    s1.first = k1 * a.first
    s2.first = k2 * a.first
    r2_add(s1, s2).first = k1 * a.first + k2 * a.first
    rhs.first = (k1 + k2) * a.first
    k1 * a.first + k2 * a.first = (k1 + k2) * a.first
    r2_add(s1, s2).first = rhs.first
    r2_add(s1, s2).second = s1.second + s2.second
    s1.second = k1 * a.second
    s2.second = k2 * a.second
    r2_add(s1, s2).second = k1 * a.second + k2 * a.second
    rhs.second = (k1 + k2) * a.second
    k1 * a.second + k2 * a.second = (k1 + k2) * a.second
    r2_add(s1, s2).second = rhs.second
    r2_add(s1, s2) = rhs
    r2_add(r2_smul(k1, a), r2_smul(k2, a)) = r2_smul(k1 + k2, a)
}

/// A scalar multiple of a negation is the negation of the scalar multiple:
/// `(-k) a = -(k a)`.
theorem angle_bisector_smul_neg_scalar(k: Real, a: Pair[Real, Real]) {
    r2_smul(-k, a) = r2_neg(r2_smul(k, a))
} by {
    let sk = r2_smul(k, a)
    let sn = r2_smul(-k, a)
    r2_pair_ext(sn, r2_neg(sk))
    sn.first = (-k) * a.first
    (-k) * a.first = -(k * a.first)
    r2_neg(sk).first = -sk.first
    sk.first = k * a.first
    r2_neg(sk).first = -(k * a.first)
    sn.first = r2_neg(sk).first
    sn.second = (-k) * a.second
    (-k) * a.second = -(k * a.second)
    r2_neg(sk).second = -sk.second
    sk.second = k * a.second
    r2_neg(sk).second = -(k * a.second)
    sn.second = r2_neg(sk).second
    sn = r2_neg(sk)
    r2_smul(-k, a) = r2_neg(r2_smul(k, a))
}

/// Scalar multiplication distributes over scalar subtraction:
/// `k1 a - k2 a = (k1 - k2) a`.
theorem angle_bisector_smul_sub_scalar(k1: Real, k2: Real, a: Pair[Real, Real]) {
    r2_sub(r2_smul(k1, a), r2_smul(k2, a)) = r2_smul(k1 - k2, a)
} by {
    let s1 = r2_smul(k1, a)
    let s2 = r2_smul(k2, a)
    let rhs = r2_smul(k1 - k2, a)
    r2_pair_ext(r2_sub(s1, s2), rhs)
    r2_sub(s1, s2).first = s1.first - s2.first
    s1.first = k1 * a.first
    s2.first = k2 * a.first
    r2_sub(s1, s2).first = k1 * a.first - k2 * a.first
    rhs.first = (k1 - k2) * a.first
    k1 * a.first - k2 * a.first = (k1 - k2) * a.first
    r2_sub(s1, s2).first = rhs.first
    r2_sub(s1, s2).second = s1.second - s2.second
    s1.second = k1 * a.second
    s2.second = k2 * a.second
    r2_sub(s1, s2).second = k1 * a.second - k2 * a.second
    rhs.second = (k1 - k2) * a.second
    k1 * a.second - k2 * a.second = (k1 - k2) * a.second
    r2_sub(s1, s2).second = rhs.second
    r2_sub(s1, s2) = rhs
    r2_sub(r2_smul(k1, a), r2_smul(k2, a)) = r2_smul(k1 - k2, a)
}

// ---------------------------------------------------------------------------
// The affine decompositions of the point `d` on the side `bc`.
// ---------------------------------------------------------------------------

/// With `d = (1 - t) b + t c`, the displacement `d - a` is the affine
/// combination `(1 - t)(b - a) + t (c - a)`.
theorem angle_bisector_d_sub_a(
    a: Pair[Real, Real], b: Pair[Real, Real], c: Pair[Real, Real],
    d: Pair[Real, Real], t: Real
) {
    r2_add(r2_smul(Real.1 - t, b), r2_smul(t, c)) = d
    implies
    r2_sub(d, a) = r2_add(r2_smul(Real.1 - t, r2_sub(b, a)), r2_smul(t, r2_sub(c, a)))
} by {
    if r2_add(r2_smul(Real.1 - t, b), r2_smul(t, c)) = d {
        r2_add(r2_smul(Real.1 - t, r2_sub(b, a)), r2_smul(t, r2_sub(c, a))) =
            r2_add(
                r2_sub(r2_smul(Real.1 - t, b), r2_smul(Real.1 - t, a)),
                r2_sub(r2_smul(t, c), r2_smul(t, a)))
        r2_smul_sub_distrib(Real.1 - t, b, a)
        r2_smul(Real.1 - t, r2_sub(b, a)) = r2_sub(r2_smul(Real.1 - t, b), r2_smul(Real.1 - t, a))
        r2_smul_sub_distrib(t, c, a)
        r2_smul(t, r2_sub(c, a)) = r2_sub(r2_smul(t, c), r2_smul(t, a))
        r2_add(
            r2_sub(r2_smul(Real.1 - t, b), r2_smul(Real.1 - t, a)),
            r2_sub(r2_smul(t, c), r2_smul(t, a))) =
            r2_sub(
                r2_add(r2_smul(Real.1 - t, b), r2_smul(t, c)),
                r2_add(r2_smul(Real.1 - t, a), r2_smul(t, a)))
        r2_sub_add_pair_distrib(
            r2_smul(Real.1 - t, b), r2_smul(t, c), r2_smul(Real.1 - t, a), r2_smul(t, a))
        r2_add(
            r2_sub(r2_smul(Real.1 - t, b), r2_smul(Real.1 - t, a)),
            r2_sub(r2_smul(t, c), r2_smul(t, a))) =
            r2_sub(
                r2_add(r2_smul(Real.1 - t, b), r2_smul(t, c)),
                r2_add(r2_smul(Real.1 - t, a), r2_smul(t, a)))
        r2_add(r2_smul(Real.1 - t, b), r2_smul(t, c)) = d
        r2_add(r2_smul(Real.1 - t, r2_sub(b, a)), r2_smul(t, r2_sub(c, a))) =
            r2_sub(d, r2_add(r2_smul(Real.1 - t, a), r2_smul(t, a)))
        angle_bisector_smul_add_scalar(Real.1 - t, t, a)
        r2_add(r2_smul(Real.1 - t, a), r2_smul(t, a)) = r2_smul(Real.1 - t + t, a)
        Real.1 - t + t = Real.1
        r2_smul(Real.1 - t + t, a) = r2_smul(Real.1, a)
        r2_smul_one(a)
        r2_smul(Real.1, a) = a
        r2_smul(Real.1 - t + t, a) = a
        r2_add(r2_smul(Real.1 - t, a), r2_smul(t, a)) = a
        r2_sub(d, r2_add(r2_smul(Real.1 - t, a), r2_smul(t, a))) = r2_sub(d, a)
        r2_add(r2_smul(Real.1 - t, r2_sub(b, a)), r2_smul(t, r2_sub(c, a))) = r2_sub(d, a)
        r2_sub(d, a) = r2_add(r2_smul(Real.1 - t, r2_sub(b, a)), r2_smul(t, r2_sub(c, a)))
    }
}

/// With `d = (1 - t) b + t c`, the displacement `d - b` is `t (c - b)`.
theorem angle_bisector_d_sub_b(
    b: Pair[Real, Real], c: Pair[Real, Real],
    d: Pair[Real, Real], t: Real
) {
    r2_add(r2_smul(Real.1 - t, b), r2_smul(t, c)) = d
    implies
    r2_sub(d, b) = r2_smul(t, r2_sub(c, b))
} by {
    if r2_add(r2_smul(Real.1 - t, b), r2_smul(t, c)) = d {
        r2_sub(d, b) = r2_sub(r2_add(r2_smul(Real.1 - t, b), r2_smul(t, c)), b)
        r2_sub_add_pair_distrib(
            r2_smul(Real.1 - t, b), r2_smul(t, c), b, Pair.new(Real.0, Real.0))
        r2_sub(
            r2_add(r2_smul(Real.1 - t, b), r2_smul(t, c)),
            r2_add(b, Pair.new(Real.0, Real.0))) =
            r2_add(
                r2_sub(r2_smul(Real.1 - t, b), b),
                r2_sub(r2_smul(t, c), Pair.new(Real.0, Real.0)))
        r2_add(b, Pair.new(Real.0, Real.0)) = b
        r2_add_zero_right(b)
        r2_sub(
            r2_add(r2_smul(Real.1 - t, b), r2_smul(t, c)),
            b) =
            r2_add(
                r2_sub(r2_smul(Real.1 - t, b), b),
                r2_sub(r2_smul(t, c), Pair.new(Real.0, Real.0)))
        angle_bisector_sub_zero_right(r2_smul(t, c))
        r2_sub(r2_smul(t, c), Pair.new(Real.0, Real.0)) = r2_smul(t, c)
        r2_sub(
            r2_add(r2_smul(Real.1 - t, b), r2_smul(t, c)),
            b) =
            r2_add(r2_sub(r2_smul(Real.1 - t, b), b), r2_smul(t, c))
        r2_smul_sub_self(Real.1 - t, b)
        r2_sub(r2_smul(Real.1 - t, b), b) = r2_smul(Real.1 - t - Real.1, b)
        Real.1 - t - Real.1 = -t
        r2_smul(Real.1 - t - Real.1, b) = r2_smul(-t, b)
        angle_bisector_smul_neg_scalar(t, b)
        r2_smul(-t, b) = r2_neg(r2_smul(t, b))
        r2_sub(r2_smul(Real.1 - t, b), b) = r2_neg(r2_smul(t, b))
        r2_add(r2_neg(r2_smul(t, b)), r2_smul(t, c)) = r2_sub(r2_smul(t, c), r2_smul(t, b))
        r2_sub_eq_add_neg(r2_smul(t, c), r2_smul(t, b))
        r2_add(r2_neg(r2_smul(t, b)), r2_smul(t, c)) = r2_sub(r2_smul(t, c), r2_smul(t, b))
        r2_sub(
            r2_add(r2_smul(Real.1 - t, b), r2_smul(t, c)),
            b) = r2_sub(r2_smul(t, c), r2_smul(t, b))
        r2_smul_sub_distrib(t, c, b)
        r2_sub(r2_smul(t, c), r2_smul(t, b)) = r2_smul(t, r2_sub(c, b))
        r2_sub(
            r2_add(r2_smul(Real.1 - t, b), r2_smul(t, c)),
            b) = r2_smul(t, r2_sub(c, b))
        r2_sub(d, b) = r2_smul(t, r2_sub(c, b))
    }
}

/// With `d = (1 - t) b + t c`, the displacement `d - c` is `(1 - t) (b - c)`.
theorem angle_bisector_d_sub_c(
    b: Pair[Real, Real], c: Pair[Real, Real],
    d: Pair[Real, Real], t: Real
) {
    r2_add(r2_smul(Real.1 - t, b), r2_smul(t, c)) = d
    implies
    r2_sub(d, c) = r2_smul(Real.1 - t, r2_sub(b, c))
} by {
    if r2_add(r2_smul(Real.1 - t, b), r2_smul(t, c)) = d {
        r2_sub(d, c) = r2_sub(r2_add(r2_smul(Real.1 - t, b), r2_smul(t, c)), c)
        r2_sub_add_pair_distrib(
            r2_smul(Real.1 - t, b), r2_smul(t, c), c, Pair.new(Real.0, Real.0))
        r2_sub(
            r2_add(r2_smul(Real.1 - t, b), r2_smul(t, c)),
            r2_add(c, Pair.new(Real.0, Real.0))) =
            r2_add(
                r2_sub(r2_smul(Real.1 - t, b), c),
                r2_sub(r2_smul(t, c), Pair.new(Real.0, Real.0)))
        r2_add_zero_right(c)
        r2_add(c, Pair.new(Real.0, Real.0)) = c
        r2_sub(
            r2_add(r2_smul(Real.1 - t, b), r2_smul(t, c)),
            c) =
            r2_add(
                r2_sub(r2_smul(Real.1 - t, b), c),
                r2_sub(r2_smul(t, c), Pair.new(Real.0, Real.0)))
        angle_bisector_sub_zero_right(r2_smul(t, c))
        r2_sub(r2_smul(t, c), Pair.new(Real.0, Real.0)) = r2_smul(t, c)
        r2_sub(
            r2_add(r2_smul(Real.1 - t, b), r2_smul(t, c)),
            c) = r2_add(r2_sub(r2_smul(Real.1 - t, b), c), r2_smul(t, c))
        r2_add_comm(r2_sub(r2_smul(Real.1 - t, b), c), r2_smul(t, c))
        r2_add(r2_sub(r2_smul(Real.1 - t, b), c), r2_smul(t, c)) =
            r2_add(r2_smul(t, c), r2_sub(r2_smul(Real.1 - t, b), c))
        r2_sub(
            r2_add(r2_smul(Real.1 - t, b), r2_smul(t, c)),
            c) = r2_add(r2_smul(t, c), r2_sub(r2_smul(Real.1 - t, b), c))
        // t c + ((1 - t) b - c) = (1 - t) b + (t c - c)
        r2_sub_eq_add_neg(c, r2_smul(t, c))
        r2_sub(c, r2_smul(t, c)) = r2_add(c, r2_neg(r2_smul(t, c)))
        r2_smul_neg(t, c)
        r2_smul(t, r2_neg(c)) = r2_neg(r2_smul(t, c))
        r2_add(r2_neg(r2_smul(t, c)), r2_sub(r2_smul(Real.1 - t, b), c)) =
            r2_sub(r2_sub(r2_smul(Real.1 - t, b), c), r2_smul(t, c))
        r2_sub_eq_add_neg(r2_sub(r2_smul(Real.1 - t, b), c), r2_smul(t, c))
        r2_add(r2_neg(r2_smul(t, c)), r2_sub(r2_smul(Real.1 - t, b), c)) =
            r2_sub(r2_sub(r2_smul(Real.1 - t, b), c), r2_smul(t, c))
        r2_add(r2_smul(t, c), r2_neg(r2_smul(t, c))) = Pair.new(Real.0, Real.0)
        r2_add_neg_right(r2_smul(t, c))
        // (1 - t) b - c - t c = (1 - t) b - (1 + t) c
        //   = (1 - t) b - (1 - t) c - (t c + t c)  [not used; direct route below]
        r2_sub(
            r2_add(r2_smul(Real.1 - t, b), r2_smul(t, c)),
            c) = r2_add(r2_smul(t, c), r2_sub(r2_smul(Real.1 - t, b), c))
        // Direct: (1 - t) b + (t c - c) with t c - c = (t - 1) c = -(1 - t) c
        r2_smul_sub_self(t, c)
        r2_sub(r2_smul(t, c), c) = r2_smul(t - Real.1, c)
        t - Real.1 = -(Real.1 - t)
        r2_smul(t - Real.1, c) = r2_smul(-(Real.1 - t), c)
        angle_bisector_smul_neg_scalar(Real.1 - t, c)
        r2_smul(-(Real.1 - t), c) = r2_neg(r2_smul(Real.1 - t, c))
        r2_sub(r2_smul(t, c), c) = r2_neg(r2_smul(Real.1 - t, c))
        // d - c = (1 - t) b + (t c - c) = (1 - t) b - (1 - t) c
        r2_sub(d, c) = r2_add(r2_smul(Real.1 - t, b), r2_sub(r2_smul(t, c), c))
        r2_sub(
            r2_add(r2_smul(Real.1 - t, b), r2_smul(t, c)),
            c) = r2_add(r2_smul(Real.1 - t, b), r2_sub(r2_smul(t, c), c))
        r2_sub(d, c) = r2_add(r2_smul(Real.1 - t, b), r2_neg(r2_smul(Real.1 - t, c)))
        r2_sub_eq_add_neg(r2_smul(Real.1 - t, b), r2_smul(Real.1 - t, c))
        r2_add(r2_smul(Real.1 - t, b), r2_neg(r2_smul(Real.1 - t, c))) =
            r2_sub(r2_smul(Real.1 - t, b), r2_smul(Real.1 - t, c))
        r2_sub(d, c) = r2_sub(r2_smul(Real.1 - t, b), r2_smul(Real.1 - t, c))
        r2_smul_sub_distrib(Real.1 - t, b, c)
        r2_sub(r2_smul(Real.1 - t, b), r2_smul(Real.1 - t, c)) =
            r2_smul(Real.1 - t, r2_sub(b, c))
        r2_sub(d, c) = r2_smul(Real.1 - t, r2_sub(b, c))
    }
}

// ---------------------------------------------------------------------------
// The cross-product expansion and the main theorem.
// ---------------------------------------------------------------------------

/// The cross product of the bisector direction with the side direction
/// expands: `cross((1 - t) U + t V, v U + u V) = ((1 - t) u - t v) cross(U, V)`.
theorem angle_bisector_cross_expand(
    p: Pair[Real, Real], q: Pair[Real, Real], t: Real, u: Real, v: Real
) {
    r2_cross(
        r2_add(r2_smul(Real.1 - t, p), r2_smul(t, q)),
        r2_add(r2_smul(v, p), r2_smul(u, q))) =
        ((Real.1 - t) * u - t * v) * r2_cross(p, q)
} by {
    r2_cross_add_left(
        r2_smul(Real.1 - t, p), r2_smul(t, q), r2_add(r2_smul(v, p), r2_smul(u, q)))
    r2_cross(
        r2_add(r2_smul(Real.1 - t, p), r2_smul(t, q)),
        r2_add(r2_smul(v, p), r2_smul(u, q))) =
        r2_cross(r2_smul(Real.1 - t, p), r2_add(r2_smul(v, p), r2_smul(u, q))) +
        r2_cross(r2_smul(t, q), r2_add(r2_smul(v, p), r2_smul(u, q)))
    r2_cross_smul_left(Real.1 - t, p, r2_add(r2_smul(v, p), r2_smul(u, q)))
    r2_cross(r2_smul(Real.1 - t, p), r2_add(r2_smul(v, p), r2_smul(u, q))) =
        (Real.1 - t) * r2_cross(p, r2_add(r2_smul(v, p), r2_smul(u, q)))
    r2_cross_smul_left(t, q, r2_add(r2_smul(v, p), r2_smul(u, q)))
    r2_cross(r2_smul(t, q), r2_add(r2_smul(v, p), r2_smul(u, q))) =
        t * r2_cross(q, r2_add(r2_smul(v, p), r2_smul(u, q)))
    r2_cross_add_right(p, r2_smul(v, p), r2_smul(u, q))
    r2_cross(p, r2_add(r2_smul(v, p), r2_smul(u, q))) =
        r2_cross(p, r2_smul(v, p)) + r2_cross(p, r2_smul(u, q))
    r2_cross_smul_right(v, p, p)
    r2_cross(p, r2_smul(v, p)) = v * r2_cross(p, p)
    r2_cross_self_zero(p)
    r2_cross(p, p) = Real.0
    v * r2_cross(p, p) = Real.0
    r2_cross(p, r2_smul(v, p)) = Real.0
    r2_cross_smul_right(u, p, q)
    r2_cross(p, r2_smul(u, q)) = u * r2_cross(p, q)
    r2_cross(p, r2_add(r2_smul(v, p), r2_smul(u, q))) = Real.0 + u * r2_cross(p, q)
    Real.0 + u * r2_cross(p, q) = u * r2_cross(p, q)
    r2_cross(p, r2_add(r2_smul(v, p), r2_smul(u, q))) = u * r2_cross(p, q)
    r2_cross_add_right(q, r2_smul(v, p), r2_smul(u, q))
    r2_cross(q, r2_add(r2_smul(v, p), r2_smul(u, q))) =
        r2_cross(q, r2_smul(v, p)) + r2_cross(q, r2_smul(u, q))
    r2_cross_smul_right(v, q, p)
    r2_cross(q, r2_smul(v, p)) = v * r2_cross(q, p)
    r2_cross_swap(q, p)
    r2_cross(q, p) = -r2_cross(p, q)
    v * r2_cross(q, p) = -(v * r2_cross(p, q))
    r2_cross(q, r2_smul(v, p)) = -(v * r2_cross(p, q))
    r2_cross_smul_right(u, q, q)
    r2_cross(q, r2_smul(u, q)) = u * r2_cross(q, q)
    r2_cross_self_zero(q)
    r2_cross(q, q) = Real.0
    u * r2_cross(q, q) = Real.0
    r2_cross(q, r2_smul(u, q)) = Real.0
    r2_cross(q, r2_add(r2_smul(v, p), r2_smul(u, q))) =
        -(v * r2_cross(p, q)) + Real.0
    -(v * r2_cross(p, q)) + Real.0 = -(v * r2_cross(p, q))
    r2_cross(q, r2_add(r2_smul(v, p), r2_smul(u, q))) = -(v * r2_cross(p, q))
    r2_cross(
        r2_add(r2_smul(Real.1 - t, p), r2_smul(t, q)),
        r2_add(r2_smul(v, p), r2_smul(u, q))) =
        (Real.1 - t) * (u * r2_cross(p, q)) + t * (-(v * r2_cross(p, q)))
    (Real.1 - t) * (u * r2_cross(p, q)) = ((Real.1 - t) * u) * r2_cross(p, q)
    t * (-(v * r2_cross(p, q))) = -(t * (v * r2_cross(p, q)))
    t * (v * r2_cross(p, q)) = (t * v) * r2_cross(p, q)
    -(t * (v * r2_cross(p, q))) = -((t * v) * r2_cross(p, q))
    ((Real.1 - t) * u) * r2_cross(p, q) + -((t * v) * r2_cross(p, q)) =
        ((Real.1 - t) * u - t * v) * r2_cross(p, q)
    r2_cross(
        r2_add(r2_smul(Real.1 - t, p), r2_smul(t, q)),
        r2_add(r2_smul(v, p), r2_smul(u, q))) =
        ((Real.1 - t) * u - t * v) * r2_cross(p, q)
}

/// A nonzero scalar times a point difference equals the point difference:
/// from `(1 - t) u = t v`, squaring gives `(1 - t)^2 u^2 = t^2 v^2`.
theorem angle_bisector_square_ratio(alpha: Real, u: Real, v: Real) {
    (Real.1 - alpha) * u = alpha * v
    implies
    (Real.1 - alpha) * (Real.1 - alpha) * (u * u) = alpha * alpha * (v * v)
} by {
    if (Real.1 - alpha) * u = alpha * v {
        ((Real.1 - alpha) * u) * ((Real.1 - alpha) * u) = (alpha * v) * (alpha * v)
        ((Real.1 - alpha) * u) * ((Real.1 - alpha) * u) =
            (Real.1 - alpha) * (Real.1 - alpha) * (u * u)
        (alpha * v) * (alpha * v) = alpha * alpha * (v * v)
        (Real.1 - alpha) * (Real.1 - alpha) * (u * u) = alpha * alpha * (v * v)
    }
}

/// The angle bisector theorem: if `d` lies on `bc` with parameter `t`,
/// `u^2 = |b - a|^2`, `v^2 = |c - a|^2`, the direction of `ad` is the
/// internal bisector direction `v (b - a) + u (c - a)`, and the triangle is
/// non-degenerate, then `|b - d|^2 v^2 = |d - c|^2 u^2`, i.e.
/// `(BD/DC)^2 = (AB/AC)^2`.
theorem theorems1000_angle_bisector(
    a: Pair[Real, Real], b: Pair[Real, Real], c: Pair[Real, Real],
    d: Pair[Real, Real], u: Real, v: Real, t: Real
) {
    u * u = r2_norm_sq(r2_sub(b, a)) and
    v * v = r2_norm_sq(r2_sub(c, a)) and
    r2_add(r2_smul(Real.1 - t, b), r2_smul(t, c)) = d and
    r2_cross(r2_sub(d, a),
        r2_add(r2_smul(v, r2_sub(b, a)), r2_smul(u, r2_sub(c, a)))) = Real.0 and
    r2_cross(r2_sub(b, a), r2_sub(c, a)) != Real.0
    implies
    r2_norm_sq(r2_sub(b, d)) * v * v = r2_norm_sq(r2_sub(d, c)) * u * u
} by {
    if u * u = r2_norm_sq(r2_sub(b, a)) and
        v * v = r2_norm_sq(r2_sub(c, a)) and
        r2_add(r2_smul(Real.1 - t, b), r2_smul(t, c)) = d and
        r2_cross(r2_sub(d, a),
            r2_add(r2_smul(v, r2_sub(b, a)), r2_smul(u, r2_sub(c, a)))) = Real.0 and
        r2_cross(r2_sub(b, a), r2_sub(c, a)) != Real.0 {
        u * u = r2_norm_sq(r2_sub(b, a))
        v * v = r2_norm_sq(r2_sub(c, a))
        r2_add(r2_smul(Real.1 - t, b), r2_smul(t, c)) = d
        r2_cross(r2_sub(d, a),
            r2_add(r2_smul(v, r2_sub(b, a)), r2_smul(u, r2_sub(c, a)))) = Real.0
        r2_cross(r2_sub(b, a), r2_sub(c, a)) != Real.0
        // d - a = (1 - t)(b - a) + t (c - a)
        angle_bisector_d_sub_a(a, b, c, d, t)
        r2_sub(d, a) = r2_add(r2_smul(Real.1 - t, r2_sub(b, a)), r2_smul(t, r2_sub(c, a)))
        angle_bisector_cross_expand(r2_sub(b, a), r2_sub(c, a), t, u, v)
        r2_cross(
            r2_add(r2_smul(Real.1 - t, r2_sub(b, a)), r2_smul(t, r2_sub(c, a))),
            r2_add(r2_smul(v, r2_sub(b, a)), r2_smul(u, r2_sub(c, a)))) =
            ((Real.1 - t) * u - t * v) * r2_cross(r2_sub(b, a), r2_sub(c, a))
        r2_cross(r2_sub(d, a),
            r2_add(r2_smul(v, r2_sub(b, a)), r2_smul(u, r2_sub(c, a)))) =
            ((Real.1 - t) * u - t * v) * r2_cross(r2_sub(b, a), r2_sub(c, a))
        ((Real.1 - t) * u - t * v) * r2_cross(r2_sub(b, a), r2_sub(c, a)) = Real.0
        // cancel the nonzero cross product
        r2_cross(r2_sub(b, a), r2_sub(c, a)) *
            ((Real.1 - t) * u - t * v) = Real.0
        r2_cross(r2_sub(b, a), r2_sub(c, a)) *
            ((Real.1 - t) * u - t * v) = Real.0 * Real.1
        mul_left_cancel(Real.0, r2_cross(r2_sub(b, a), r2_sub(c, a)),
            (Real.1 - t) * u - t * v)
        Real.0 / r2_cross(r2_sub(b, a), r2_sub(c, a)) =
            (Real.1 - t) * u - t * v
        Real.0 / r2_cross(r2_sub(b, a), r2_sub(c, a)) = Real.0
        (Real.1 - t) * u - t * v = Real.0
        (Real.1 - t) * u = t * v
        angle_bisector_square_ratio(t, u, v)
        (Real.1 - t) * (Real.1 - t) * (u * u) = t * t * (v * v)
        // |b - d|^2 = t^2 |c - b|^2
        angle_bisector_d_sub_b(b, c, d, t)
        r2_sub(d, b) = r2_smul(t, r2_sub(c, b))
        r2_norm_sq_smul(t, r2_sub(c, b))
        r2_norm_sq(r2_smul(t, r2_sub(c, b))) = t * t * r2_norm_sq(r2_sub(c, b))
        r2_norm_sq(r2_sub(d, b)) = t * t * r2_norm_sq(r2_sub(c, b))
        r2_sub_reverse_neg(b, d)
        r2_sub(b, d) = r2_neg(r2_sub(d, b))
        r2_norm_sq_neg(r2_sub(d, b))
        r2_norm_sq(r2_neg(r2_sub(d, b))) = r2_norm_sq(r2_sub(d, b))
        r2_norm_sq(r2_sub(b, d)) = r2_norm_sq(r2_sub(d, b))
        r2_sub_reverse_neg(c, b)
        r2_sub(c, b) = r2_neg(r2_sub(b, c))
        r2_norm_sq_neg(r2_sub(b, c))
        r2_norm_sq(r2_neg(r2_sub(b, c))) = r2_norm_sq(r2_sub(b, c))
        r2_norm_sq(r2_sub(c, b)) = r2_norm_sq(r2_sub(b, c))
        r2_norm_sq(r2_sub(b, d)) = t * t * r2_norm_sq(r2_sub(b, c))
        // |d - c|^2 = (1 - t)^2 |b - c|^2
        angle_bisector_d_sub_c(b, c, d, t)
        r2_sub(d, c) = r2_smul(Real.1 - t, r2_sub(b, c))
        r2_norm_sq_smul(Real.1 - t, r2_sub(b, c))
        r2_norm_sq(r2_smul(Real.1 - t, r2_sub(b, c))) =
            (Real.1 - t) * (Real.1 - t) * r2_norm_sq(r2_sub(b, c))
        r2_norm_sq(r2_sub(d, c)) =
            (Real.1 - t) * (Real.1 - t) * r2_norm_sq(r2_sub(b, c))
        // the ratio identity
        r2_norm_sq(r2_sub(b, d)) * (v * v) =
            (t * t * r2_norm_sq(r2_sub(b, c))) * (v * v)
        r2_norm_sq(r2_sub(d, c)) * (u * u) =
            ((Real.1 - t) * (Real.1 - t) * r2_norm_sq(r2_sub(b, c))) * (u * u)
        (t * t * r2_norm_sq(r2_sub(b, c))) * (v * v) =
            r2_norm_sq(r2_sub(b, c)) * (t * t * (v * v))
        ((Real.1 - t) * (Real.1 - t) * r2_norm_sq(r2_sub(b, c))) * (u * u) =
            r2_norm_sq(r2_sub(b, c)) * ((Real.1 - t) * (Real.1 - t) * (u * u))
        (Real.1 - t) * (Real.1 - t) * (u * u) = t * t * (v * v)
        r2_norm_sq(r2_sub(b, c)) * ((Real.1 - t) * (Real.1 - t) * (u * u)) =
            r2_norm_sq(r2_sub(b, c)) * (t * t * (v * v))
        (t * t * r2_norm_sq(r2_sub(b, c))) * (v * v) =
            ((Real.1 - t) * (Real.1 - t) * r2_norm_sq(r2_sub(b, c))) * (u * u)
        r2_norm_sq(r2_sub(b, d)) * (v * v) = r2_norm_sq(r2_sub(d, c)) * (u * u)
        r2_norm_sq(r2_sub(b, d)) * v * v = r2_norm_sq(r2_sub(d, c)) * u * u
    }
}
