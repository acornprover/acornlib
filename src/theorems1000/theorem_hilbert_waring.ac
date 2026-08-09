from nat import Nat
from list import partial

numerals Nat

// The Hilbert-Waring theorem: for every exponent `k >= 1` there is a
// least number `g(k)` such that every natural number is the sum of at most
// `g(k)` nonnegative `k`-th powers.  David Hilbert proved the existence of
// `g(k)` for all `k` in 1909, answering a question posed by Edward
// Waring in 1770.  The known exact values begin with `g(1) = 1`,
// `g(2) = 4` (Lagrange's four-square theorem, stated in
// theorem_lagrange_four_squares.ac in this directory), `g(3) = 9`,
// `g(4) = 19`, and `g(5) = 37`.
//
// The statement below uses the list partial sum `partial(w, g)`, the sum
// of `w(0) + ... + w(g-1)`: a witness function `w: Nat -> Nat` with
// `n = partial(w, g)` expresses `n` as the sum of the `g` summands
// `w(0), ..., w(g-1)`, and the condition that each summand is a `k`-th
// power is `exists(x: Nat) { w(i) = x.pow(k) }`.
//
// Proof sketch: the classical proof (Hilbert 1909) is an identity-based
// induction on `k`: if `n` is a sum of `s` `k`-th powers and `N` is a
// fixed large multiple of `n` expressed as a sum of `r` `(k+1)`-th powers
// (using the "Hilbert identity" polynomial identity), then `n N` is a sum
// of `s r` `(k+1)`-th powers, giving a bound for sums of `(k+1)`-th
// powers.  The base case `k = 2` is Lagrange's four-square theorem, whose
// proof (via Euler's four-square identity and a descent argument) is
// outlined in theorem_lagrange_four_squares.ac.  The Hilbert identity and
// the descent for the four-square theorem are not yet in the library.
//
// theorem theorems1000_hilbert_waring {
//     forall(k: Nat) {
//         Nat.1 <= k implies
//         exists(g: Nat) {
//             forall(n: Nat) {
//                 exists(w: Nat -> Nat) {
//                     n = partial(w, g) and
//                     forall(i: Nat) {
//                         i < g implies exists(x: Nat) { w(i) = x.pow(k) }
//                     }
//                 }
//             }
//         }
//     }
// }
