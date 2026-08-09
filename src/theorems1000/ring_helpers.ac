from real import Real
from theorems1000.r2_helpers import real_add_pair_rearrange

numerals Real

// Shared scalar ring-identity helpers for the theorems1000 algebra
// theorems (Sophie Germain's identity, the Brahmagupta-Fibonacci identity,
// Lagrange's identity, Euler's four-square identity).  The helpers provide
// the expansion of squares of sums and differences, and the cancellation
// of opposite middle terms in a sum of expanded squares.  Each helper is
// a small, explicitly proved step so that the larger identities can be
// assembled as chains of citations.

/// The square of a two-term sum.
theorem ring_square_add(x: Real, y: Real) {
    (x + y) * (x + y) = x * x + y * x + (x * y + y * y)
} by {
    (x + y) * (x + y) = (x + y) * x + (x + y) * y
    (x + y) * x = x * x + y * x
    (x + y) * y = x * y + y * y
    (x + y) * (x + y) = x * x + y * x + (x * y + y * y)
}

/// The square of a difference.
theorem ring_square_sub(x: Real, y: Real) {
    (x - y) * (x - y) = x * x - x * y - y * x + y * y
} by {
    (x - y) * (x - y) = (x - y) * x - (x - y) * y
    (x - y) * x = x * x - y * x
    (x - y) * y = x * y - y * y
    (x - y) * (x - y) = x * x - y * x - (x * y - y * y)
    y * x = x * y
    x * x - y * x - (x * y - y * y) = x * x - x * y - (x * y - y * y)
    x * x - x * y - (x * y - y * y) = x * x - x * y + -(x * y - y * y)
    -(x * y - y * y) = -(x * y) + y * y
    x * x - x * y + -(x * y - y * y) = x * x - x * y + (-(x * y) + y * y)
    x * x - x * y + (-(x * y) + y * y) = x * x - x * y - x * y + y * y
    x * x - x * y - x * y + y * y = x * x - x * y - y * x + y * y
}

/// A difference of three terms expands to a sum of negations.
theorem ring_minus_to_add(x: Real, m: Real, n: Real, y: Real) {
    x - m - n + y = x + -m + -n + y
}

/// A sum of negations regroups as a difference of sums.
theorem ring_add_to_minus(x: Real, m: Real, n: Real, y: Real) {
    x + -m + -n + y = (x - m) + (y - n)
} by {
    x + -m + -n + y = (x + -m) + (-n + y)
    (x + -m) + (-n + y) = (x - m) + (y - n)
}

/// Cancelling a term and its negation: `x - m + (m + z) = x + z`.
theorem ring_cancel_sub_add(x: Real, m: Real, z: Real) {
    x - m + (m + z) = x + z
} by {
    x - m = x + -m
    x + -m + (m + z) = x + (-m + (m + z))
    -m + (m + z) = (-m + m) + z
    -m + m = Real.0
    (-m + m) + z = Real.0 + z
    Real.0 + z = z
    x + (-m + (m + z)) = x + z
}

/// Cancelling a swapped term and its negation: `x - m + (z + m) = x + z`.
theorem ring_cancel_swap_add(x: Real, m: Real, z: Real) {
    x - m + (z + m) = x + z
} by {
    z + m = m + z
    x - m + (z + m) = x - m + (m + z)
    ring_cancel_sub_add(x, m, z)
    x - m + (m + z) = x + z
}

/// Opposite middle terms cancel in a sum of two expanded squares:
/// `(x - m - n + y) + (z + m + n + w) = x + y + z + w`.
theorem ring_cancel_middle(x: Real, y: Real, z: Real, w: Real, m: Real, n: Real) {
    (x - m - n + y) + (z + m + n + w) = x + y + z + w
} by {
    ring_minus_to_add(x, m, n, y)
    x - m - n + y = x + -m + -n + y
    ring_add_to_minus(x, m, n, y)
    x + -m + -n + y = (x - m) + (y - n)
    z + m + n + w = (z + m) + (n + w)
    real_add_pair_rearrange(x - m, y - n, z + m, n + w)
    ((x - m) + (y - n)) + ((z + m) + (n + w)) =
        ((x - m) + (z + m)) + ((y - n) + (n + w))
    ring_cancel_swap_add(x, m, z)
    x - m + (z + m) = x + z
    ring_cancel_sub_add(y, n, w)
    y - n + (n + w) = y + w
    ((x - m) + (z + m)) + ((y - n) + (n + w)) = (x + z) + (y + w)
}

/// Cancelling a term against its negation: `x + m + (z + -m) = x + z`.
theorem ring_cancel_add_sub(x: Real, m: Real, z: Real) {
    x + m + (z + -m) = x + z
} by {
    x + m + (z + -m) = x + (m + (z + -m))
    m + (z + -m) = (m + z) + -m
    (m + z) + -m = (z + m) + -m
    z + m = m + z
    (z + m) + -m = z + (m + -m)
    m + -m = Real.0
    z + (m + -m) = z + Real.0
    x + ((z + m) + -m) = x + (z + Real.0)
    x + (z + Real.0) = x + z
    x + (m + (z + -m)) = x + z
}

/// Cancelling a swapped term against its negation: `x + m + (-m + z) = x + z`.
theorem ring_cancel_swap_sub(x: Real, m: Real, z: Real) {
    x + m + (-m + z) = x + z
} by {
    -m + z = z + -m
    x + m + (-m + z) = x + m + (z + -m)
    ring_cancel_add_sub(x, m, z)
    x + m + (z + -m) = x + z
}

/// Opposite middle terms cancel in a sum of two expanded squares, with the
/// plus-form on the left: `(x + m + n + y) + (z - m - n + w) = x + y + z + w`.
theorem ring_cancel_middle_plus(x: Real, y: Real, z: Real, w: Real, m: Real, n: Real) {
    (x + m + n + y) + (z - m - n + w) = x + y + z + w
} by {
    ring_minus_to_add(z, m, n, w)
    z - m - n + w = z + -m + -n + w
    x + m + n + y = (x + m) + (n + y)
    z + -m + -n + w = (z + -m) + (-n + w)
    real_add_pair_rearrange(x + m, n + y, z + -m, -n + w)
    ((x + m) + (n + y)) + ((z + -m) + (-n + w)) =
        ((x + m) + (z + -m)) + ((n + y) + (-n + w))
    ring_cancel_add_sub(x, m, z)
    x + m + (z + -m) = x + z
    ring_cancel_swap_sub(y, n, w)
    y + n + (-n + w) = y + w
    ((x + m) + (z + -m)) + ((n + y) + (-n + w)) = (x + z) + (y + w)
}
