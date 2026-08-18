from nat import Nat, from_nat
from real import Real, converges_to

// The prime number theorem (conjectured by Adrien-Marie Legendre 1797 and
// Carl Friedrich Gauss 1792; proved independently by Jacques Hadamard and
// Charles Jean de la Vallée Poussin in 1896, with elementary proofs by
// Paul Erdős and Atle Selberg in 1949): the number `π(x)` of primes not
// exceeding `x` is asymptotically `x / ln x`:
//     π(x) ~ x / ln x    as x → infinity,
// i.e. the ratio `π(x) · ln x / x` tends to 1.  The theorem is the crown
// of analytic number theory: it says that the density of primes near `x`
// is about `1 / ln x`, so the nth prime `p_n` satisfies `p_n ~ n ln n`.
// The classical proof passes through the Riemann zeta function (Hadamard
// and de la Vallée Poussin proved that `ζ(s) ≠ 0` on the line
// `Re s = 1`, which is the analytic input); the elementary proof of
// Erdős–Selberg uses the identity of Selberg.  The error term is
// controlled by the Riemann hypothesis: the statement
// `π(x) = li(x) + O(√x ln x)` (with the logarithmic integral `li`) is
// equivalent to the Riemann hypothesis.
//
// The formal statement below uses the library's sequence notation, as in
// `theorem_stirling.ac`: `converges_to(q, a)` is the Weierstrass
// convergence of a real sequence to `a`, `from_nat[Real](n)` embeds the
// natural `n` into the reals, and `x.log.get_or_else(Real.0)` is the natural
// logarithm.  The counting function `π` is described rather than defined:
// the library has the prime predicate `is_prime` on the naturals but no
// prime-counting function, so `prime_count` below stands for the number
// of primes in `{1, ..., n}`; a faithful definition would be a recursive
// count over the range `[1, n]`.
//
// Proof sketch (analytic route): the theorem is equivalent to the absence
// of zeros of the Riemann zeta function on the line `Re s = 1`.  The
// proof needs the zeta function, its Euler product, analytic
// continuation, and the Wiener–Ikehara Tauberian theorem (or an
// equivalent contour argument); none of this machinery is present in the
// library, so the statement is recorded without proof.  The entry is the
// "Prime number theorem" of the "1000+ theorems" list.

// theorem theorems1000_prime_number {
//     converges_to(
//         function(k: Nat) {
//             let n: Nat = k + Nat.1
//             from_nat[Real](prime_count(n)) * (from_nat[Real](n)).log.get_or_else(Real.0) /
//                 from_nat[Real](n)
//         },
//         Real.1)
// }
