from pair import Pair
from real import Real
from top100 import r2_sub, r2_norm_sq

// The Steiner–Lehmus theorem: if two internal angle bisectors of a triangle
// have equal lengths, then the triangle is isosceles.  (If the bisectors of
// the angles at `b` and `c` — meeting the opposite sides at `e` and `f` —
// satisfy `|b - e| = |c - f|`, then `|a - b| = |a - c|`.)
//
// As in `theorems1000_angle_bisector` in this directory, the bisector
// condition is recorded through the angle-bisector theorem: the internal
// bisector from `b` meets `ac` at the point `e` dividing the side in the
// ratio of the adjacent sides,
//     |e - a|^2 * |b - c|^2 = |e - c|^2 * |b - a|^2,
// and cyclically the bisector from `c` meets `ab` at `f` with
//     |f - a|^2 * |c - b|^2 = |f - b|^2 * |c - a|^2.
// The equal-bisector hypothesis is `|b - e|^2 = |c - f|^2`, and the
// conclusion is `|b - a|^2 = |c - a|^2`.
//
// Proof sketch: the classical proof applies the angle-bisector theorem in
// both directions.  Write `x = |a - b|`, `y = |a - c|`, `z = |b - c|` for
// the side lengths.  The bisector length formula (from the angle-bisector
// theorem and the law of cosines) gives
//     |b - e|^2 = x z (1 - (y / (x + z))^2)   and cyclically
//     |c - f|^2 = y z (1 - (x / (y + z))^2),
// and equating the two bisector lengths yields, after clearing the positive
// denominators,
//     (x - y) (x y (x + y) + z (x^2 + x y + y^2) + z^2 (x + y) + ...) = 0,
// whose second factor is strictly positive, forcing `x = y`.  The standard
// proof instead compares the two half-angle sines: `|b - e| = |c - f|`
// implies `(B/2).sin = (C/2).sin` from the sine law in the two small
// triangles, and since `B/2, C/2` lie in `(0, pi/2)` the sines are equal
// only for equal angles, so `B = C` and the triangle is isosceles.  Both
// routes need the sine law and the angle-bisector length formula, which are
// not yet in the library; the statement is therefore recorded with the
// bisector conditions in squared-ratio form and the proof left as a sketch.
//
// theorem theorems1000_steiner_lehmus(
//     a: Pair[Real, Real], b: Pair[Real, Real], c: Pair[Real, Real],
//     e: Pair[Real, Real], f: Pair[Real, Real]
// ) {
//     r2_norm_sq(r2_sub(e, a)) * r2_norm_sq(r2_sub(b, c)) =
//         r2_norm_sq(r2_sub(e, c)) * r2_norm_sq(r2_sub(b, a)) and
//     r2_norm_sq(r2_sub(f, a)) * r2_norm_sq(r2_sub(c, b)) =
//         r2_norm_sq(r2_sub(f, b)) * r2_norm_sq(r2_sub(c, a)) and
//     r2_norm_sq(r2_sub(b, e)) = r2_norm_sq(r2_sub(c, f))
//     implies
//     r2_norm_sq(r2_sub(b, a)) = r2_norm_sq(r2_sub(c, a))
// }
