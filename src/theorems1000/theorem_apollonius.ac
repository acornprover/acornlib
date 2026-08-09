from pair import Pair
from real import Real
from top100 import r2_add, r2_neg, r2_sub, r2_dot, r2_norm_sq
from theorems1000.r2_helpers import r2_sub_through, r2_midpoint_sub_neg, r2_parallelogram_law, r2_sub_eq_add_neg

// Apollonius's theorem: the sum of the squares of two sides of a triangle
// equals twice the square of the median to the third side plus twice the
// square of the half-side.  Here `m` is the midpoint of the side `bc`, so
// `b + c = 2m`, and the statement is
//     |b - a|^2 + |c - a|^2 = 2 * (|m - a|^2 + |b - m|^2).

/// The displacement `c - a` through the midpoint is the difference of the
/// two half-median displacements.
theorem apollonius_c_minus_a(
    a: Pair[Real, Real], b: Pair[Real, Real], c: Pair[Real, Real], m: Pair[Real, Real]
) {
    r2_add(b, c) = r2_add(m, m) implies
    r2_sub(c, a) = r2_sub(r2_sub(m, a), r2_sub(b, m))
} by {
    if r2_add(b, c) = r2_add(m, m) {
        r2_midpoint_sub_neg(b, c, m)
        r2_sub(c, m) = r2_neg(r2_sub(b, m))
        r2_sub_through(a, m, c)
        r2_sub(c, a) = r2_add(r2_sub(m, a), r2_sub(c, m))
        r2_sub(c, a) = r2_add(r2_sub(m, a), r2_neg(r2_sub(b, m)))
        r2_sub_eq_add_neg(r2_sub(m, a), r2_sub(b, m))
        r2_sub(r2_sub(m, a), r2_sub(b, m)) = r2_add(r2_sub(m, a), r2_neg(r2_sub(b, m)))
        r2_sub(c, a) = r2_sub(r2_sub(m, a), r2_sub(b, m))
    }
}

/// Apollonius's theorem for a triangle with median to `bc` at `m`.
///
/// With `m` the midpoint of `bc`, the sum of the squared lengths of the two
/// sides through `a` equals twice the squared median plus twice the squared
/// half-side: `|b-a|^2 + |c-a|^2 = 2(|m-a|^2 + |b-m|^2)`.
theorem theorems1000_apollonius(
    a: Pair[Real, Real], b: Pair[Real, Real], c: Pair[Real, Real], m: Pair[Real, Real]
) {
    r2_add(b, c) = r2_add(m, m) implies
    r2_norm_sq(r2_sub(b, a)) + r2_norm_sq(r2_sub(c, a)) =
        (Real.1 + Real.1) * (r2_norm_sq(r2_sub(m, a)) + r2_norm_sq(r2_sub(b, m)))
} by {
    if r2_add(b, c) = r2_add(m, m) {
        // u = m - a, v = b - m
        // b - a = u + v
        r2_sub_through(a, m, b)
        r2_sub(b, a) = r2_add(r2_sub(m, a), r2_sub(b, m))
        // c - a = u - v
        apollonius_c_minus_a(a, b, c, m)
        r2_sub(c, a) = r2_sub(r2_sub(m, a), r2_sub(b, m))
        // |u + v|^2 + |u - v|^2 = 2|u|^2 + 2|v|^2
        r2_parallelogram_law(r2_sub(m, a), r2_sub(b, m))
        r2_norm_sq(r2_add(r2_sub(m, a), r2_sub(b, m))) + r2_norm_sq(r2_sub(r2_sub(m, a), r2_sub(b, m))) =
            (Real.1 + Real.1) * r2_norm_sq(r2_sub(m, a)) + (Real.1 + Real.1) * r2_norm_sq(r2_sub(b, m))
        r2_norm_sq(r2_sub(b, a)) = r2_norm_sq(r2_add(r2_sub(m, a), r2_sub(b, m)))
        r2_norm_sq(r2_sub(c, a)) = r2_norm_sq(r2_sub(r2_sub(m, a), r2_sub(b, m)))
        r2_norm_sq(r2_sub(b, a)) + r2_norm_sq(r2_sub(c, a)) =
            r2_norm_sq(r2_add(r2_sub(m, a), r2_sub(b, m))) + r2_norm_sq(r2_sub(r2_sub(m, a), r2_sub(b, m)))
        r2_norm_sq(r2_sub(b, a)) + r2_norm_sq(r2_sub(c, a)) =
            (Real.1 + Real.1) * r2_norm_sq(r2_sub(m, a)) + (Real.1 + Real.1) * r2_norm_sq(r2_sub(b, m))
        (Real.1 + Real.1) * r2_norm_sq(r2_sub(m, a)) + (Real.1 + Real.1) * r2_norm_sq(r2_sub(b, m)) =
            (Real.1 + Real.1) * (r2_norm_sq(r2_sub(m, a)) + r2_norm_sq(r2_sub(b, m)))
        r2_norm_sq(r2_sub(b, a)) + r2_norm_sq(r2_sub(c, a)) =
            (Real.1 + Real.1) * (r2_norm_sq(r2_sub(m, a)) + r2_norm_sq(r2_sub(b, m)))
    }
}
