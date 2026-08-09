from pair import Pair
from real import Real
from top100 import r2_add, r2_sub, r2_dot, r2_norm_sq
from theorems1000.r2_helpers import r2_smul, r2_sub_through, r2_norm_sq_add_expansion,
    r2_norm_sq_smul, r2_dot_smul_right, r2_add_sub_left_cancel

// Stewart's theorem: in a triangle, the length of a cevian is determined by
// the lengths of the three sides and the two segments into which the cevian
// divides the opposite side.  With `d` on the side `bc` dividing it as
// `bd = t * bc` and `dc = (1 - t) * bc`, the classical statement
//     |ca|^2 * bd + |ab|^2 * dc = |bc| * (|ad|^2 + bd * dc)
// becomes, after cancelling the common factor `|bc|`,
//     t * |c-a|^2 + (1 - t) * |b-a|^2 = |d-a|^2 + t * (1 - t) * |c-b|^2,
// which is exactly the weighted-mean identity for the squared distance from
// `a` to the point `d = (1 - t) b + t c` on the segment `bc`.  Everything is
// written with squared distances, so no square roots are needed.  For
// `t = 1/2` this reduces to Apollonius's theorem (the median case).

/// The scalar identity behind Stewart's theorem: expanding
/// `t |u + v|^2 + (1 - t) |u|^2` and `|u + t v|^2 + t (1 - t) |v|^2`
/// gives the same combination of `u2 = |u|^2`, `v2 = |v|^2` and
/// `uv = u . v`.

/// Expanding `t * (u2 + v2 + uv + uv)` distributes over the four terms.
theorem real_stewart_expand(u2: Real, v2: Real, uv: Real, t: Real) {
    t * (u2 + v2 + uv + uv) = t * u2 + t * v2 + t * uv + t * uv
} by {
    u2 + v2 + uv + uv = (u2 + v2) + (uv + uv)
    t * (u2 + v2 + uv + uv) = t * ((u2 + v2) + (uv + uv))
    t * ((u2 + v2) + (uv + uv)) = t * (u2 + v2) + t * (uv + uv)
    t * (u2 + v2) = t * u2 + t * v2
    t * (uv + uv) = t * uv + t * uv
    t * (u2 + v2 + uv + uv) = t * u2 + t * v2 + t * uv + t * uv
}

/// The terms `t u2` and `(1 - t) u2` combine back to `u2`.
theorem real_stewart_cancel(u2: Real, v2: Real, uv: Real, t: Real) {
    (t * u2 + t * v2 + t * uv + t * uv) + (Real.1 - t) * u2 =
        u2 + t * v2 + t * uv + t * uv
} by {
    (Real.1 - t) * u2 = u2 - t * u2
    (t * u2 + t * v2 + t * uv + t * uv) + (u2 - t * u2) =
        t * u2 + t * v2 + t * uv + t * uv + u2 + -(t * u2)
    t * u2 + t * v2 + t * uv + t * uv + u2 + -(t * u2) =
        t * u2 + t * v2 + t * uv + -(t * u2) + t * uv + u2
    t * u2 + t * v2 + t * uv + -(t * u2) + t * uv + u2 =
        t * u2 + t * v2 + -(t * u2) + t * uv + t * uv + u2
    t * u2 + t * v2 + -(t * u2) + t * uv + t * uv + u2 =
        t * u2 + -(t * u2) + t * v2 + t * uv + t * uv + u2
    t * u2 + -(t * u2) + t * v2 + t * uv + t * uv + u2 =
        t * u2 + -(t * u2) + t * v2 + t * uv + u2 + t * uv
    t * u2 + -(t * u2) + t * v2 + t * uv + u2 + t * uv =
        t * u2 + -(t * u2) + t * v2 + u2 + t * uv + t * uv
    t * u2 + -(t * u2) + t * v2 + u2 + t * uv + t * uv =
        (t * u2 + -(t * u2)) + (t * v2 + u2) + (t * uv + t * uv)
    (t * u2 + -(t * u2)) + (t * v2 + u2) + (t * uv + t * uv) =
        u2 + t * v2 + t * uv + t * uv
    (t * u2 + t * v2 + t * uv + t * uv) + (Real.1 - t) * u2 =
        u2 + t * v2 + t * uv + t * uv
}

/// The splitting step `t = t^2 + t (1 - t)` multiplied by `v2`.
theorem real_stewart_split(t: Real, v2: Real) {
    t * v2 = (t * t) * v2 + t * (Real.1 - t) * v2
} by {
    (t * t) * v2 + t * (Real.1 - t) * v2 = t * (t + (Real.1 - t)) * v2
    t + (Real.1 - t) = (Real.1 - t) + t
    t * (t + (Real.1 - t)) * v2 = t * ((Real.1 - t) + t) * v2
    t * ((Real.1 - t) + t) * v2 = t * Real.1 * v2
    t * Real.1 * v2 = t * v2
    (t * t) * v2 + t * (Real.1 - t) * v2 = t * v2
}

/// The full scalar identity, proved by chaining the three pieces.
theorem real_stewart_scalar(u2: Real, v2: Real, uv: Real, t: Real) {
    t * (u2 + v2 + uv + uv) + (Real.1 - t) * u2 =
        u2 + (t * t) * v2 + t * uv + t * uv + t * (Real.1 - t) * v2
} by {
    real_stewart_expand(u2, v2, uv, t)
    t * (u2 + v2 + uv + uv) = t * u2 + t * v2 + t * uv + t * uv
    real_stewart_cancel(u2, v2, uv, t)
    (t * u2 + t * v2 + t * uv + t * uv) + (Real.1 - t) * u2 =
        u2 + t * v2 + t * uv + t * uv
    t * (u2 + v2 + uv + uv) + (Real.1 - t) * u2 =
        (t * u2 + t * v2 + t * uv + t * uv) + (Real.1 - t) * u2
    t * (u2 + v2 + uv + uv) + (Real.1 - t) * u2 =
        u2 + t * v2 + t * uv + t * uv
    real_stewart_split(t, v2)
    t * v2 = (t * t) * v2 + t * (Real.1 - t) * v2
    u2 + t * v2 + t * uv + t * uv =
        u2 + (t * t) * v2 + t * (Real.1 - t) * v2 + t * uv + t * uv
    u2 + (t * t) * v2 + t * (Real.1 - t) * v2 + t * uv + t * uv =
        u2 + (t * t) * v2 + t * uv + t * (Real.1 - t) * v2 + t * uv
    u2 + (t * t) * v2 + t * uv + t * (Real.1 - t) * v2 + t * uv =
        u2 + (t * t) * v2 + t * uv + t * uv + t * (Real.1 - t) * v2
    t * (u2 + v2 + uv + uv) + (Real.1 - t) * u2 =
        u2 + (t * t) * v2 + t * uv + t * uv + t * (Real.1 - t) * v2
}

/// Stewart's theorem for a triangle with a cevian to `bc` at `d`.
///
/// With `d = b + t (c - b)` on the side `bc`, the squared length of the
/// cevian `ad` satisfies
///     |d-a|^2 = t * |c-a|^2 + (1 - t) * |b-a|^2 - t * (1 - t) * |c-b|^2,
/// equivalently `|ca|^2 * bd + |ab|^2 * dc = |bc| * (|ad|^2 + bd * dc)`.
theorem theorems1000_stewart(
    a: Pair[Real, Real], b: Pair[Real, Real], c: Pair[Real, Real],
    d: Pair[Real, Real], t: Real
) {
    r2_add(b, r2_smul(t, r2_sub(c, b))) = d implies
    t * r2_norm_sq(r2_sub(c, a)) + (Real.1 - t) * r2_norm_sq(r2_sub(b, a)) =
        r2_norm_sq(r2_sub(d, a)) + t * (Real.1 - t) * r2_norm_sq(r2_sub(c, b))
} by {
    if r2_add(b, r2_smul(t, r2_sub(c, b))) = d {
        r2_add(b, r2_smul(t, r2_sub(c, b))) = d
        // d - b = t (c - b)
        r2_add_sub_left_cancel(b, r2_smul(t, r2_sub(c, b)))
        r2_sub(r2_add(b, r2_smul(t, r2_sub(c, b))), b) = r2_smul(t, r2_sub(c, b))
        r2_sub(d, b) = r2_smul(t, r2_sub(c, b))
        // d - a = (b - a) + t (c - b)
        r2_sub_through(a, b, d)
        r2_sub(d, a) = r2_add(r2_sub(b, a), r2_sub(d, b))
        r2_sub(d, a) = r2_add(r2_sub(b, a), r2_smul(t, r2_sub(c, b)))
        // c - a = (b - a) + (c - b)
        r2_sub_through(a, b, c)
        r2_sub(c, a) = r2_add(r2_sub(b, a), r2_sub(c, b))
        // |d - a|^2 = |b-a|^2 + t^2 |c-b|^2 + 2 t (b-a).(c-b)
        r2_norm_sq_add_expansion(r2_sub(b, a), r2_smul(t, r2_sub(c, b)))
        r2_norm_sq(r2_add(r2_sub(b, a), r2_smul(t, r2_sub(c, b)))) =
            r2_norm_sq(r2_sub(b, a)) + r2_norm_sq(r2_smul(t, r2_sub(c, b))) +
            r2_dot(r2_sub(b, a), r2_smul(t, r2_sub(c, b))) +
            r2_dot(r2_sub(b, a), r2_smul(t, r2_sub(c, b)))
        r2_norm_sq(r2_sub(d, a)) =
            r2_norm_sq(r2_sub(b, a)) + r2_norm_sq(r2_smul(t, r2_sub(c, b))) +
            r2_dot(r2_sub(b, a), r2_smul(t, r2_sub(c, b))) +
            r2_dot(r2_sub(b, a), r2_smul(t, r2_sub(c, b)))
        r2_norm_sq_smul(t, r2_sub(c, b))
        r2_norm_sq(r2_smul(t, r2_sub(c, b))) = t * t * r2_norm_sq(r2_sub(c, b))
        r2_dot_smul_right(t, r2_sub(b, a), r2_sub(c, b))
        r2_dot(r2_sub(b, a), r2_smul(t, r2_sub(c, b))) = t * r2_dot(r2_sub(b, a), r2_sub(c, b))
        r2_norm_sq(r2_sub(d, a)) =
            r2_norm_sq(r2_sub(b, a)) + t * t * r2_norm_sq(r2_sub(c, b)) +
            t * r2_dot(r2_sub(b, a), r2_sub(c, b)) + t * r2_dot(r2_sub(b, a), r2_sub(c, b))
        // |c - a|^2 = |b-a|^2 + |c-b|^2 + 2 (b-a).(c-b)
        r2_norm_sq_add_expansion(r2_sub(b, a), r2_sub(c, b))
        r2_norm_sq(r2_add(r2_sub(b, a), r2_sub(c, b))) =
            r2_norm_sq(r2_sub(b, a)) + r2_norm_sq(r2_sub(c, b)) +
            r2_dot(r2_sub(b, a), r2_sub(c, b)) + r2_dot(r2_sub(b, a), r2_sub(c, b))
        r2_norm_sq(r2_sub(c, a)) =
            r2_norm_sq(r2_sub(b, a)) + r2_norm_sq(r2_sub(c, b)) +
            r2_dot(r2_sub(b, a), r2_sub(c, b)) + r2_dot(r2_sub(b, a), r2_sub(c, b))
        // the scalar identity links the two expansions
        real_stewart_scalar(
            r2_norm_sq(r2_sub(b, a)),
            r2_norm_sq(r2_sub(c, b)),
            r2_dot(r2_sub(b, a), r2_sub(c, b)),
            t)
        t * (r2_norm_sq(r2_sub(b, a)) + r2_norm_sq(r2_sub(c, b)) +
                r2_dot(r2_sub(b, a), r2_sub(c, b)) + r2_dot(r2_sub(b, a), r2_sub(c, b))) +
            (Real.1 - t) * r2_norm_sq(r2_sub(b, a)) =
            r2_norm_sq(r2_sub(b, a)) + (t * t) * r2_norm_sq(r2_sub(c, b)) +
            t * r2_dot(r2_sub(b, a), r2_sub(c, b)) + t * r2_dot(r2_sub(b, a), r2_sub(c, b)) +
            t * (Real.1 - t) * r2_norm_sq(r2_sub(c, b))
        // the left side of the goal expands to the scalar identity's left side
        t * r2_norm_sq(r2_sub(c, a)) + (Real.1 - t) * r2_norm_sq(r2_sub(b, a)) =
            t * (r2_norm_sq(r2_sub(b, a)) + r2_norm_sq(r2_sub(c, b)) +
                r2_dot(r2_sub(b, a), r2_sub(c, b)) + r2_dot(r2_sub(b, a), r2_sub(c, b))) +
            (Real.1 - t) * r2_norm_sq(r2_sub(b, a))
        // the right side of the goal expands to the scalar identity's right side
        r2_norm_sq(r2_sub(d, a)) + t * (Real.1 - t) * r2_norm_sq(r2_sub(c, b)) =
            r2_norm_sq(r2_sub(b, a)) + (t * t) * r2_norm_sq(r2_sub(c, b)) +
            t * r2_dot(r2_sub(b, a), r2_sub(c, b)) + t * r2_dot(r2_sub(b, a), r2_sub(c, b)) +
            t * (Real.1 - t) * r2_norm_sq(r2_sub(c, b))
        t * r2_norm_sq(r2_sub(c, a)) + (Real.1 - t) * r2_norm_sq(r2_sub(b, a)) =
            r2_norm_sq(r2_sub(d, a)) + t * (Real.1 - t) * r2_norm_sq(r2_sub(c, b))
    }
}
