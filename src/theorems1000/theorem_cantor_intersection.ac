from nat import Nat
from real import Real, converges, limit, is_increasing, is_upper_bound,
    monotone_convergence_principle, increasing_convergent_bounded_by_limit,
    eventual_ub, ub_imp_limit_lte
from order import is_monotone, is_antitone, monotone_apply, antitone_apply,
    lt_imp_lte, lte_trans
from order_cases import gt_of_not_lte

numerals Real

// Cantor's intersection theorem (Georg Cantor, 1872, in his proof of the
// uniqueness of trigonometric series expansions): a nested sequence of
// closed intervals of real numbers whose lengths shrink to zero has a
// common point.  The general form proved here drops the shrinking-length
// hypothesis and asserts only nonemptiness of the intersection: if
//     a(0) <= a(1) <= a(2) <= ... <= b(2) <= b(1) <= b(0),
// i.e. the left endpoints form a nondecreasing sequence, the right
// endpoints a nonincreasing sequence, and each interval `[a(n), b(n)]` is
// well-formed (`a(n) <= b(n)`), then some real `x` lies in every interval.
// When the lengths `b(n) - a(n)` shrink to zero the common point is unique,
// and the theorem is then a form of the completeness of the reals: it is
// equivalent to the existence of suprema, and it underlies the Bolzano-
// Weierstrass theorem (a bounded sequence has a convergent subsequence),
// the construction of the real numbers from nested intervals, and the
// existence of the real root of any continuous sign change.
//
// Proof: the left endpoints `a(n)` form a nondecreasing sequence bounded
// above by `b(0)`, so by the monotone convergence principle it converges,
// and its limit `l = limit(a)` is at least every `a(n)`.  Each right
// endpoint `b(m)` bounds every left endpoint (`a(n) <= b(m)` for all
// `n, m`: for `n <= m` chain `a(n) <= a(m) <= b(m)`, for `m <= n` chain
// `a(n) <= b(n) <= b(m)`), so `b(m)` is an upper bound of the sequence and
// `l <= b(m)` by `ub_imp_limit_lte`.  Hence `l` lies in every
// `[a(n), b(n)]`.
//
// Nondecreasing is written with the order-theoretic predicate
// `is_monotone(a)`, which is the pointwise condition `a(x) <= a(y)` for
// `x <= y` and can be applied directly between arbitrary indices (the
// library's `is_increasing` is the same notion, but from outside the real
// package its pointwise form is not usable as a hypothesis); `is_antitone
// (b)` is the reversed condition `b(y) <= b(x)` for `x <= y`.

/// A nondecreasing sequence is increasing in the pointwise sense used by
/// the monotone convergence principle.
theorem theorems1000_monotone_is_increasing(a: Nat -> Real) {
    is_monotone(a) implies is_increasing(a)
} by {
    if is_monotone(a) {
        forall(n: Nat) {
            n <= n.suc
            monotone_apply(a, n, n.suc)
            a(n) <= a(n.suc)
        }
        is_increasing(a)
    }
}

/// Every left endpoint is below every right endpoint of a nested family of
/// intervals.
theorem theorems1000_nested_cross_bound(a: Nat -> Real, b: Nat -> Real, n: Nat, m: Nat) {
    is_monotone(a) and is_antitone(b) and (forall(k: Nat) { a(k) <= b(k) })
    implies a(n) <= b(m)
} by {
    if is_monotone(a) and is_antitone(b) and (forall(k: Nat) { a(k) <= b(k) }) {
        if n <= m {
            monotone_apply(a, n, m)
            a(n) <= a(m)
            a(m) <= b(m)
            lte_trans(a(n), a(m), b(m))
            a(n) <= b(m)
        }
        if not n <= m {
            gt_of_not_lte(n, m)
            not n <= m implies n > m
            n > m
            m < n
            lt_imp_lte(m, n)
            m <= n
            antitone_apply(b, m, n)
            b(n) <= b(m)
            a(n) <= b(n)
            lte_trans(a(n), b(n), b(m))
            a(n) <= b(m)
        }
        a(n) <= b(m)
    }
}

/// Cantor's intersection theorem: a nested sequence of closed intervals
/// with nondecreasing left endpoints and nonincreasing right endpoints has
/// a common point.
theorem theorems1000_cantor_intersection(a: Nat -> Real, b: Nat -> Real) {
    is_monotone(a) and is_antitone(b) and (forall(k: Nat) { a(k) <= b(k) })
    implies exists(x: Real) { forall(n: Nat) { a(n) <= x and x <= b(n) } }
} by {
    if is_monotone(a) and is_antitone(b) and (forall(k: Nat) { a(k) <= b(k) }) {
        // The left endpoints are bounded above by b(0).
        forall(n: Nat) {
            theorems1000_nested_cross_bound(a, b, n, Nat.0)
            a(n) <= b(Nat.0)
        }
        is_upper_bound(a, b(Nat.0))
        // By monotone convergence the left endpoints converge, and every
        // left endpoint is below the limit.
        theorems1000_monotone_is_increasing(a)
        is_increasing(a)
        monotone_convergence_principle(a, b(Nat.0))
        converges(a)
        increasing_convergent_bounded_by_limit(a)
        is_upper_bound(a, limit(a))
        forall(n: Nat) {
            a(n) <= limit(a)
        }
        // Each right endpoint is an upper bound of the left endpoints, so
        // the limit is below every right endpoint.
        forall(m: Nat) {
            forall(n: Nat) {
                theorems1000_nested_cross_bound(a, b, n, m)
                a(n) <= b(m)
            }
            is_upper_bound(a, b(m))
            (eventual_ub(a, b(m)) = exists(n0: Nat) {
                forall(i: Nat) {
                    n0 <= i implies a(i) <= b(m)
                }
            })
            exists(n0: Nat) {
                forall(i: Nat) {
                    n0 <= i implies a(i) <= b(m)
                }
            }
            eventual_ub(a, b(m))
            ub_imp_limit_lte(a, b(m))
            limit(a) <= b(m)
        }
        // The limit is the common point.
        forall(n: Nat) {
            a(n) <= limit(a) and limit(a) <= b(n)
        }
        exists(x: Real) { forall(n: Nat) { a(n) <= x and x <= b(n) } }
    }
}
