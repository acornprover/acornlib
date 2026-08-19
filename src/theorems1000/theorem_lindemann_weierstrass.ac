from nat import Nat, from_nat
from real import Real

// The Lindemann-Weierstrass theorem (1885): if alpha_1, ..., alpha_n are
// algebraic numbers that are linearly independent over the rationals, then
// the exponentials e^{alpha_1}, ..., e^{alpha_n} are algebraically
// independent over the rationals.  In particular, taking n = 1:
//     for every nonzero algebraic number alpha, e^alpha is transcendental,
// which contains the classical results
//     e is transcendental (Hermite, 1873),
//     pi is transcendental (Lindemann, 1882),
// the second because i pi would otherwise give e^{i pi} = -1 algebraic,
// contradicting transcendence of e^{i pi}.  From pi transcendental, the
// classical "squaring the circle" problem is impossible: the compass-and-
// straightedge constructible numbers are algebraic, so the circle of
// radius 1 has area pi, which is not constructible.
//
// The library does not yet define the algebraic numbers or transcendence
// over the rationals, so the theorem is recorded here in words and as the
// formal corollary that the exponential of a natural number is not a root
// of a polynomial with natural coefficients — the closest statement that
// the current library can express for "e^alpha is transcendental".  A full
// statement needs a formal theory of algebraic numbers (the algebraic
// closure of the rationals) and of linear/algebraic independence, together
// with the analytic core of the proof: the symmetric-function estimates of
// Hermite's interpolation polynomials, which are far beyond the current
// library.
//
// theorem theorems1000_lindemann_weierstrass {
//     forall(m: Nat, a: Nat) {
//         Nat.0 < a and Nat.0 < m implies
//             not exists(c: Nat -> Nat) {
//                 c(m) != Nat.0 and
//                 partial(function(k: Nat) { c(k) * (from_nat[Real](a)).exp.pow(k) }, m + Nat.1) =
//                     Real.0
//             }
//     }
// }
