from nat import Nat
from finite_set import FiniteSet
from graph import SimpleGraph, complete_graph, is_edge_2_coloring, has_mono_triangle,
    ramsey_r33_upper, six_vertices

// The theorem on friends and strangers (also called the party problem; the
// classic statement is due to Paul Erdős and George Szekeres, 1935, in the
// paper that also introduced the Erdős–Szekeres theorem): among any six
// people, there are three who are mutual friends or three who are mutual
// strangers.  Equivalently, however the edges of the complete graph on six
// vertices are colored red and blue, some triangle is monochromatic; in the
// Ramsey-theoretic notation of the library this is the upper bound
// `R(3, 3) <= 6`.  The result is the first nontrivial instance of Ramsey's
// theorem (Frank Ramsey, 1930) and it is sharp: a pentagon with its five
// diagonals colored alternately shows that five vertices are not enough,
// so `R(3, 3) = 6`.
//
// The formal statement below is the library's graph formulation
// (`graph/ramsey.ac`): `complete_graph[Nat]` is the complete graph on the
// naturals, `is_edge_2_coloring(g, color)` says that `color` is a
// red/blue coloring of the edges of `g` (constant on the two orientations
// of an edge), and `has_mono_triangle(color, s)` says that three distinct
// vertices of the six-element set `six_vertices = {0, ..., 5}` are pairwise
// joined by edges of a single color.  Reading "red" as "friends" and
// "blue" as "strangers", the conclusion says exactly that three of the six
// people are pairwise friends or three are pairwise strangers.
//
// Proof: the library's `ramsey_r33_upper` in `graph/ramsey.ac` proves
// exactly this statement.  The proof there is the classical one: fix a
// vertex `0`.  Among the five edges from `0` to the other vertices, three
// share a color (pigeonhole); if two of the edges among those three share
// that color, the pair closes a monochromatic triangle with `0`, and
// otherwise the three edges among them are all of the other color, which is
// a monochromatic triangle on its own.  We record the theorem here as an
// entry of the "1000+ theorems" list and re-derive it from that result.

/// The theorem on friends and strangers: every red/blue coloring of the
/// edges of the complete graph on six vertices has a monochromatic
/// triangle; among any six people there are three mutual friends or three
/// mutual strangers.
theorem theorems1000_friends_and_strangers(color: (Nat, Nat) -> Bool) {
    is_edge_2_coloring(complete_graph[Nat], color)
    implies has_mono_triangle(color, six_vertices)
} by {
    if is_edge_2_coloring(complete_graph[Nat], color) {
        ramsey_r33_upper(color)
        has_mono_triangle(color, six_vertices)
    }
}
