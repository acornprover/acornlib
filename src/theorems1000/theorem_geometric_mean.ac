from pair import Pair
from real import Real
from top100 import r2_add, r2_neg, r2_sub, r2_dot, r2_norm_sq
from theorems1000.r2_helpers import r2_sub_through, r2_sub_reverse_neg, r2_add_comm,
    r2_sub_eq_add_neg, r2_add_sub_left_cancel, r2_add_sub_rearrange, r2_sub_sub_same_base, r2_dot_add_right,
    r2_dot_sub_left, r2_dot_sub_right, r2_dot_neg_left, r2_dot_smul_left,
    r2_dot_smul_right, r2_norm_sq_sub_expansion, r2_norm_sq_add_expansion,
    r2_norm_sq_smul, r2_norm_sq_neg, r2_smul, r2_smul_sub_distrib, r2_smul_complement

// The geometric mean theorem: in a right triangle, the altitude to the
// hypotenuse is the geometric mean of the two segments into which it divides
// the hypotenuse.  In squared form, with `a` the right-angle vertex and `f`
// the foot of the altitude from `a` onto the hypotenuse `bc`, the identity
// reads
//     |a-f|^2 * |a-f|^2 = |b-f|^2 * |f-c|^2,
// i.e. the squared altitude is the geometric mean of the squared segments.
// The point `f` is the foot when it lies on `bc` (here `f = b + t(c-b)` for
// the parameter `t`) and the altitude is perpendicular to the hypotenuse
// (`(a-f) . (c-b) = 0`); the right angle at `a` is `(b-a) . (c-a) = 0`.

/// The complement parameter sums to one: (1 - t) + t = 1.
theorem real_geom_complement(t: Real) {
    (Real.1 - t) + t = Real.1
}

/// The two complementary pieces of the squared altitude assemble to t(1-t)S.
theorem real_geom_factor(t: Real, s: Real) {
    (Real.1 - t) * ((Real.1 - t) * (t * s)) + (t * t) * ((Real.1 - t) * s) =
        t * (Real.1 - t) * s
} by {
    (Real.1 - t) * ((Real.1 - t) * (t * s)) = (Real.1 - t) * (Real.1 - t) * (t * s)
    (t * t) * ((Real.1 - t) * s) = (t * t) * (Real.1 - t) * s
    (Real.1 - t) * (Real.1 - t) * (t * s) = t * (Real.1 - t) * ((Real.1 - t) * s)
    (t * t) * (Real.1 - t) * s = t * (Real.1 - t) * (t * s)
    t * (Real.1 - t) * ((Real.1 - t) * s) + t * (Real.1 - t) * (t * s) =
        t * (Real.1 - t) * (((Real.1 - t) * s) + (t * s))
    ((Real.1 - t) * s) + (t * s) = ((Real.1 - t) + t) * s
    t * (Real.1 - t) * (((Real.1 - t) * s) + (t * s)) = t * (Real.1 - t) * (((Real.1 - t) + t) * s)
    real_geom_complement(t)
    (Real.1 - t) + t = Real.1
    t * (Real.1 - t) * (((Real.1 - t) + t) * s) = t * (Real.1 - t) * (Real.1 * s)
    t * (Real.1 - t) * (Real.1 * s) = t * (Real.1 - t) * s
}

/// Squaring the geometric-mean product: (t(1-t)S)^2 = t^2 (1-t)^2 S^2.
theorem real_geom_square(t: Real, u: Real, v: Real) {
    u = t * (Real.1 - t) * v implies
    u * u = (t * t) * ((Real.1 - t) * (Real.1 - t)) * (v * v)
} by {
    if u = t * (Real.1 - t) * v {
        u * u = (t * (Real.1 - t) * v) * (t * (Real.1 - t) * v)
        (t * (Real.1 - t) * v) * (t * (Real.1 - t) * v) =
            (t * t) * ((Real.1 - t) * (Real.1 - t)) * (v * v)
        u * u = (t * t) * ((Real.1 - t) * (Real.1 - t)) * (v * v)
    }
}

/// The product of the two squared segments is t^2 (1-t)^2 S^2.
theorem real_geom_right(t: Real, u: Real, v: Real, s: Real) {
    u = (t * t) * s and v = ((Real.1 - t) * (Real.1 - t)) * s implies
    u * v = (t * t) * ((Real.1 - t) * (Real.1 - t)) * (s * s)
} by {
    if u = (t * t) * s and v = ((Real.1 - t) * (Real.1 - t)) * s {
        u * v = ((t * t) * s) * (((Real.1 - t) * (Real.1 - t)) * s)
        ((t * t) * s) * (((Real.1 - t) * (Real.1 - t)) * s) =
            (t * t) * (s * (((Real.1 - t) * (Real.1 - t)) * s))
        s * (((Real.1 - t) * (Real.1 - t)) * s) =
            (Real.1 - t) * (Real.1 - t) * (s * s)
        (t * t) * (s * (((Real.1 - t) * (Real.1 - t)) * s)) =
            (t * t) * ((Real.1 - t) * (Real.1 - t)) * (s * s)
        u * v = (t * t) * ((Real.1 - t) * (Real.1 - t)) * (s * s)
    }
}

/// The geometric mean theorem for a right triangle.
///
/// In the right triangle with right angle at `a` and the foot `f` of the
/// altitude from `a` onto the hypotenuse `bc`, the squared altitude is the
/// geometric mean of the squared hypotenuse segments:
/// |a-f|^2 * |a-f|^2 = |b-f|^2 * |f-c|^2.
theorem theorems1000_geometric_mean(
    a: Pair[Real, Real], b: Pair[Real, Real], c: Pair[Real, Real],
    f: Pair[Real, Real], t: Real
) {
    r2_dot(r2_sub(b, a), r2_sub(c, a)) = Real.0 and
    r2_add(b, r2_smul(t, r2_sub(c, b))) = f and
    r2_dot(r2_sub(a, f), r2_sub(c, b)) = Real.0
    implies
    r2_norm_sq(r2_sub(a, f)) * r2_norm_sq(r2_sub(a, f)) =
        r2_norm_sq(r2_sub(b, f)) * r2_norm_sq(r2_sub(f, c))
} by {
    if r2_dot(r2_sub(b, a), r2_sub(c, a)) = Real.0 and
        r2_add(b, r2_smul(t, r2_sub(c, b))) = f and
        r2_dot(r2_sub(a, f), r2_sub(c, b)) = Real.0 {
        r2_dot(r2_sub(b, a), r2_sub(c, a)) = Real.0
        r2_add(b, r2_smul(t, r2_sub(c, b))) = f
        r2_dot(r2_sub(a, f), r2_sub(c, b)) = Real.0
        // |c-b|^2 = |b-a|^2 + |c-a|^2 (Pythagoras for the right angle at a)
        r2_sub_sub_same_base(a, b, c)
        r2_sub(r2_sub(c, a), r2_sub(b, a)) = r2_sub(c, b)
        r2_norm_sq_sub_expansion(r2_sub(b, a), r2_sub(c, a))
        r2_norm_sq(r2_sub(r2_sub(c, a), r2_sub(b, a))) =
            r2_norm_sq(r2_sub(c, a)) + r2_norm_sq(r2_sub(b, a)) -
            r2_dot(r2_sub(b, a), r2_sub(c, a)) - r2_dot(r2_sub(b, a), r2_sub(c, a))
        r2_norm_sq(r2_sub(c, b)) =
            r2_norm_sq(r2_sub(c, a)) + r2_norm_sq(r2_sub(b, a)) -
            r2_dot(r2_sub(b, a), r2_sub(c, a)) - r2_dot(r2_sub(b, a), r2_sub(c, a))
        r2_dot(r2_sub(b, a), r2_sub(c, a)) = Real.0
        r2_norm_sq(r2_sub(c, b)) =
            r2_norm_sq(r2_sub(c, a)) + r2_norm_sq(r2_sub(b, a)) - Real.0 - Real.0
        r2_norm_sq(r2_sub(c, a)) + r2_norm_sq(r2_sub(b, a)) - Real.0 - Real.0 =
            r2_norm_sq(r2_sub(c, a)) + r2_norm_sq(r2_sub(b, a))
        r2_norm_sq(r2_sub(c, a)) + r2_norm_sq(r2_sub(b, a)) =
            r2_norm_sq(r2_sub(b, a)) + r2_norm_sq(r2_sub(c, a))
        r2_norm_sq(r2_sub(c, b)) = r2_norm_sq(r2_sub(b, a)) + r2_norm_sq(r2_sub(c, a))
        // t * |c-b|^2 = |b-a|^2 from the altitude condition (a-f) . (c-b) = 0
        r2_dot_sub_right(a, f, r2_sub(c, b))
        r2_dot(r2_sub(a, f), r2_sub(c, b)) =
            r2_dot(a, r2_sub(c, b)) - r2_dot(f, r2_sub(c, b))
        r2_dot(a, r2_sub(c, b)) - r2_dot(f, r2_sub(c, b)) = Real.0
        r2_dot_add_right(b, r2_smul(t, r2_sub(c, b)), r2_sub(c, b))
        r2_dot(r2_add(b, r2_smul(t, r2_sub(c, b))), r2_sub(c, b)) =
            r2_dot(b, r2_sub(c, b)) + r2_dot(r2_smul(t, r2_sub(c, b)), r2_sub(c, b))
        r2_dot(f, r2_sub(c, b)) =
            r2_dot(b, r2_sub(c, b)) + r2_dot(r2_smul(t, r2_sub(c, b)), r2_sub(c, b))
        r2_dot_smul_left(t, r2_sub(c, b), r2_sub(c, b))
        r2_dot(r2_smul(t, r2_sub(c, b)), r2_sub(c, b)) = t * r2_dot(r2_sub(c, b), r2_sub(c, b))
        r2_dot(f, r2_sub(c, b)) =
            r2_dot(b, r2_sub(c, b)) + t * r2_dot(r2_sub(c, b), r2_sub(c, b))
        r2_dot_sub_left(a, b, r2_sub(c, b))
        r2_dot(r2_sub(a, b), r2_sub(c, b)) = r2_dot(a, r2_sub(c, b)) - r2_dot(b, r2_sub(c, b))
        r2_dot(a, r2_sub(c, b)) - r2_dot(f, r2_sub(c, b)) =
            r2_dot(a, r2_sub(c, b)) - (r2_dot(b, r2_sub(c, b)) + t * r2_dot(r2_sub(c, b), r2_sub(c, b)))
        r2_dot(a, r2_sub(c, b)) - (r2_dot(b, r2_sub(c, b)) + t * r2_dot(r2_sub(c, b), r2_sub(c, b))) =
            (r2_dot(a, r2_sub(c, b)) - r2_dot(b, r2_sub(c, b))) - t * r2_dot(r2_sub(c, b), r2_sub(c, b))
        r2_dot(a, r2_sub(c, b)) - r2_dot(b, r2_sub(c, b)) =
            r2_dot(r2_sub(a, b), r2_sub(c, b))
        r2_dot(a, r2_sub(c, b)) - r2_dot(f, r2_sub(c, b)) =
            r2_dot(r2_sub(a, b), r2_sub(c, b)) - t * r2_dot(r2_sub(c, b), r2_sub(c, b))
        r2_dot(r2_sub(a, b), r2_sub(c, b)) - t * r2_dot(r2_sub(c, b), r2_sub(c, b)) = Real.0
        // (a-b) . (c-b) = |b-a|^2
        r2_dot_sub_right(r2_sub(b, a), r2_sub(c, a), r2_sub(b, a))
        r2_dot(r2_sub(b, a), r2_sub(c, b)) =
            r2_dot(r2_sub(b, a), r2_sub(c, a)) - r2_dot(r2_sub(b, a), r2_sub(b, a))
        r2_dot(r2_sub(b, a), r2_sub(c, b)) = Real.0 - r2_dot(r2_sub(b, a), r2_sub(b, a))
        Real.0 - r2_dot(r2_sub(b, a), r2_sub(b, a)) = -r2_dot(r2_sub(b, a), r2_sub(b, a))
        r2_dot(r2_sub(b, a), r2_sub(b, a)) = r2_norm_sq(r2_sub(b, a))
        r2_dot(r2_sub(b, a), r2_sub(c, b)) = -r2_norm_sq(r2_sub(b, a))
        r2_dot_neg_left(r2_sub(a, b), r2_sub(c, b))
        r2_dot(r2_neg(r2_sub(a, b)), r2_sub(c, b)) = -r2_dot(r2_sub(a, b), r2_sub(c, b))
        r2_sub_reverse_neg(b, a)
        r2_sub(b, a) = r2_neg(r2_sub(a, b))
        r2_dot(r2_sub(b, a), r2_sub(c, b)) = -r2_dot(r2_sub(a, b), r2_sub(c, b))
        -r2_dot(r2_sub(a, b), r2_sub(c, b)) = -r2_norm_sq(r2_sub(b, a))
        r2_dot(r2_sub(a, b), r2_sub(c, b)) = r2_norm_sq(r2_sub(b, a))
        // |b-a|^2 - t |c-b|^2 = 0
        r2_norm_sq(r2_sub(b, a)) - t * r2_dot(r2_sub(c, b), r2_sub(c, b)) = Real.0
        r2_dot(r2_sub(c, b), r2_sub(c, b)) = r2_norm_sq(r2_sub(c, b))
        r2_norm_sq(r2_sub(b, a)) - t * r2_norm_sq(r2_sub(c, b)) = Real.0
        r2_norm_sq(r2_sub(b, a)) = t * r2_norm_sq(r2_sub(c, b))
        // (1-t) * |c-b|^2 = |c-a|^2
        (Real.1 - t) * r2_norm_sq(r2_sub(c, b)) =
            r2_norm_sq(r2_sub(c, b)) - t * r2_norm_sq(r2_sub(c, b))
        r2_norm_sq(r2_sub(c, b)) - t * r2_norm_sq(r2_sub(c, b)) =
            r2_norm_sq(r2_sub(c, b)) - r2_norm_sq(r2_sub(b, a))
        r2_norm_sq(r2_sub(c, b)) - r2_norm_sq(r2_sub(b, a)) = r2_norm_sq(r2_sub(c, a))
        (Real.1 - t) * r2_norm_sq(r2_sub(c, b)) = r2_norm_sq(r2_sub(c, a))
        // |b - f|^2 = t^2 |c - b|^2
        r2_add_sub_left_cancel(b, r2_smul(t, r2_sub(c, b)))
        r2_sub(r2_add(b, r2_smul(t, r2_sub(c, b))), b) = r2_smul(t, r2_sub(c, b))
        r2_sub(f, b) = r2_smul(t, r2_sub(c, b))
        r2_sub_reverse_neg(b, f)
        r2_sub(b, f) = r2_neg(r2_sub(f, b))
        r2_sub(b, f) = r2_neg(r2_smul(t, r2_sub(c, b)))
        r2_norm_sq_neg(r2_smul(t, r2_sub(c, b)))
        r2_norm_sq(r2_neg(r2_smul(t, r2_sub(c, b)))) = r2_norm_sq(r2_smul(t, r2_sub(c, b)))
        r2_norm_sq(r2_sub(b, f)) = r2_norm_sq(r2_smul(t, r2_sub(c, b)))
        r2_norm_sq_smul(t, r2_sub(c, b))
        r2_norm_sq(r2_smul(t, r2_sub(c, b))) = t * t * r2_norm_sq(r2_sub(c, b))
        r2_norm_sq(r2_sub(b, f)) = t * t * r2_norm_sq(r2_sub(c, b))
        // |f - c|^2 = (1-t)^2 |c - b|^2
        r2_sub_through(f, b, c)
        r2_sub(c, f) = r2_add(r2_sub(b, f), r2_sub(c, b))
        r2_sub(c, f) = r2_add(r2_neg(r2_smul(t, r2_sub(c, b))), r2_sub(c, b))
        r2_add_comm(r2_neg(r2_smul(t, r2_sub(c, b))), r2_sub(c, b))
        r2_sub(c, f) = r2_add(r2_sub(c, b), r2_neg(r2_smul(t, r2_sub(c, b))))
        r2_sub_eq_add_neg(r2_sub(c, b), r2_smul(t, r2_sub(c, b)))
        r2_sub(r2_sub(c, b), r2_smul(t, r2_sub(c, b))) =
            r2_add(r2_sub(c, b), r2_neg(r2_smul(t, r2_sub(c, b))))
        r2_sub(c, f) = r2_sub(r2_sub(c, b), r2_smul(t, r2_sub(c, b)))
        r2_smul_complement(t, r2_sub(c, b))
        r2_sub(r2_sub(c, b), r2_smul(t, r2_sub(c, b))) = r2_smul(Real.1 - t, r2_sub(c, b))
        r2_sub(c, f) = r2_smul(Real.1 - t, r2_sub(c, b))
        r2_sub_reverse_neg(c, f)
        r2_sub(c, f) = r2_neg(r2_sub(f, c))
        r2_sub(f, c) = r2_neg(r2_smul(Real.1 - t, r2_sub(c, b)))
        r2_norm_sq_neg(r2_smul(Real.1 - t, r2_sub(c, b)))
        r2_norm_sq(r2_neg(r2_smul(Real.1 - t, r2_sub(c, b)))) =
            r2_norm_sq(r2_smul(Real.1 - t, r2_sub(c, b)))
        r2_norm_sq(r2_sub(f, c)) = r2_norm_sq(r2_smul(Real.1 - t, r2_sub(c, b)))
        r2_norm_sq_smul(Real.1 - t, r2_sub(c, b))
        r2_norm_sq(r2_smul(Real.1 - t, r2_sub(c, b))) =
            (Real.1 - t) * (Real.1 - t) * r2_norm_sq(r2_sub(c, b))
        r2_norm_sq(r2_sub(f, c)) =
            (Real.1 - t) * (Real.1 - t) * r2_norm_sq(r2_sub(c, b))
        // |a - f|^2 = (1-t)^2 |b-a|^2 + t^2 |c-a|^2
        r2_sub_through(a, b, f)
        r2_sub(f, a) = r2_add(r2_sub(b, a), r2_sub(f, b))
        r2_sub(f, a) = r2_add(r2_sub(b, a), r2_smul(t, r2_sub(c, b)))
        r2_smul_sub_distrib(t, r2_sub(c, a), r2_sub(b, a))
        r2_smul(t, r2_sub(r2_sub(c, a), r2_sub(b, a))) =
            r2_sub(r2_smul(t, r2_sub(c, a)), r2_smul(t, r2_sub(b, a)))
        r2_sub(c, b) = r2_sub(r2_sub(c, a), r2_sub(b, a))
        r2_smul(t, r2_sub(c, b)) = r2_sub(r2_smul(t, r2_sub(c, a)), r2_smul(t, r2_sub(b, a)))
        r2_sub(f, a) =
            r2_add(r2_sub(b, a), r2_sub(r2_smul(t, r2_sub(c, a)), r2_smul(t, r2_sub(b, a))))
        r2_add_sub_rearrange(r2_sub(b, a), r2_smul(t, r2_sub(c, a)), r2_smul(t, r2_sub(b, a)))
        r2_add(r2_sub(b, a), r2_sub(r2_smul(t, r2_sub(c, a)), r2_smul(t, r2_sub(b, a)))) =
            r2_add(r2_sub(r2_sub(b, a), r2_smul(t, r2_sub(b, a))), r2_smul(t, r2_sub(c, a)))
        r2_sub(f, a) =
            r2_add(r2_sub(r2_sub(b, a), r2_smul(t, r2_sub(b, a))), r2_smul(t, r2_sub(c, a)))
        r2_smul_complement(t, r2_sub(b, a))
        r2_sub(r2_sub(b, a), r2_smul(t, r2_sub(b, a))) = r2_smul(Real.1 - t, r2_sub(b, a))
        r2_sub(f, a) = r2_add(r2_smul(Real.1 - t, r2_sub(b, a)), r2_smul(t, r2_sub(c, a)))
        r2_norm_sq_add_expansion(r2_smul(Real.1 - t, r2_sub(b, a)), r2_smul(t, r2_sub(c, a)))
        r2_norm_sq(r2_add(r2_smul(Real.1 - t, r2_sub(b, a)), r2_smul(t, r2_sub(c, a)))) =
            r2_norm_sq(r2_smul(Real.1 - t, r2_sub(b, a))) + r2_norm_sq(r2_smul(t, r2_sub(c, a))) +
            r2_dot(r2_smul(Real.1 - t, r2_sub(b, a)), r2_smul(t, r2_sub(c, a))) +
            r2_dot(r2_smul(Real.1 - t, r2_sub(b, a)), r2_smul(t, r2_sub(c, a)))
        r2_norm_sq(r2_sub(f, a)) =
            r2_norm_sq(r2_smul(Real.1 - t, r2_sub(b, a))) + r2_norm_sq(r2_smul(t, r2_sub(c, a))) +
            r2_dot(r2_smul(Real.1 - t, r2_sub(b, a)), r2_smul(t, r2_sub(c, a))) +
            r2_dot(r2_smul(Real.1 - t, r2_sub(b, a)), r2_smul(t, r2_sub(c, a)))
        r2_norm_sq_smul(Real.1 - t, r2_sub(b, a))
        r2_norm_sq(r2_smul(Real.1 - t, r2_sub(b, a))) =
            (Real.1 - t) * (Real.1 - t) * r2_norm_sq(r2_sub(b, a))
        r2_norm_sq_smul(t, r2_sub(c, a))
        r2_norm_sq(r2_smul(t, r2_sub(c, a))) = t * t * r2_norm_sq(r2_sub(c, a))
        r2_dot_smul_left(Real.1 - t, r2_sub(b, a), r2_smul(t, r2_sub(c, a)))
        r2_dot(r2_smul(Real.1 - t, r2_sub(b, a)), r2_smul(t, r2_sub(c, a))) =
            (Real.1 - t) * r2_dot(r2_sub(b, a), r2_smul(t, r2_sub(c, a)))
        r2_dot_smul_right(t, r2_sub(b, a), r2_sub(c, a))
        r2_dot(r2_sub(b, a), r2_smul(t, r2_sub(c, a))) = t * r2_dot(r2_sub(b, a), r2_sub(c, a))
        r2_dot(r2_sub(b, a), r2_sub(c, a)) = Real.0
        t * r2_dot(r2_sub(b, a), r2_sub(c, a)) = Real.0
        (Real.1 - t) * (t * r2_dot(r2_sub(b, a), r2_sub(c, a))) = Real.0
        r2_dot(r2_smul(Real.1 - t, r2_sub(b, a)), r2_smul(t, r2_sub(c, a))) = Real.0
        r2_norm_sq(r2_sub(f, a)) =
            (Real.1 - t) * (Real.1 - t) * r2_norm_sq(r2_sub(b, a)) +
            t * t * r2_norm_sq(r2_sub(c, a)) + Real.0 + Real.0
        r2_norm_sq(r2_sub(f, a)) =
            (Real.1 - t) * (Real.1 - t) * r2_norm_sq(r2_sub(b, a)) +
            t * t * r2_norm_sq(r2_sub(c, a))
        // assemble: |a-f|^2 = t(1-t) |c-b|^2
        r2_norm_sq(r2_sub(f, a)) =
            (Real.1 - t) * ((Real.1 - t) * (t * r2_norm_sq(r2_sub(c, b)))) +
            (t * t) * ((Real.1 - t) * r2_norm_sq(r2_sub(c, b)))
        real_geom_factor(t, r2_norm_sq(r2_sub(c, b)))
        (Real.1 - t) * ((Real.1 - t) * (t * r2_norm_sq(r2_sub(c, b)))) +
            (t * t) * ((Real.1 - t) * r2_norm_sq(r2_sub(c, b))) =
            t * (Real.1 - t) * r2_norm_sq(r2_sub(c, b))
        r2_norm_sq(r2_sub(f, a)) = t * (Real.1 - t) * r2_norm_sq(r2_sub(c, b))
        // (|a-f|^2)^2 = t^2 (1-t)^2 |c-b|^4
        real_geom_square(t, r2_norm_sq(r2_sub(f, a)), r2_norm_sq(r2_sub(c, b)))
        r2_norm_sq(r2_sub(f, a)) * r2_norm_sq(r2_sub(f, a)) =
            (t * t) * ((Real.1 - t) * (Real.1 - t)) * (r2_norm_sq(r2_sub(c, b)) * r2_norm_sq(r2_sub(c, b)))
        // |b-f|^2 * |f-c|^2 = t^2 (1-t)^2 |c-b|^4
        real_geom_right(t, r2_norm_sq(r2_sub(b, f)), r2_norm_sq(r2_sub(f, c)), r2_norm_sq(r2_sub(c, b)))
        r2_norm_sq(r2_sub(b, f)) * r2_norm_sq(r2_sub(f, c)) =
            (t * t) * ((Real.1 - t) * (Real.1 - t)) * (r2_norm_sq(r2_sub(c, b)) * r2_norm_sq(r2_sub(c, b)))
        r2_norm_sq(r2_sub(a, f)) * r2_norm_sq(r2_sub(a, f)) =
            r2_norm_sq(r2_sub(b, f)) * r2_norm_sq(r2_sub(f, c))
    }
}
