from pair import Pair
from real import Real, lt_swap_neg, lt_add_right, lt_add_left, lt_trans
from top100 import r2_dot, r2_sub, r2_norm_sq
from theorems1000.r2_helpers import r2_norm_sq_sub_expansion

// The hinge theorem: if two sides of one triangle are equal to two sides of
// another triangle, and the included angle of the first triangle is larger
// than the included angle of the second, then the third side of the first
// triangle is longer than the third side of the second.
//
// The two triangles are given by the side vectors `u`, `v` and `u'`, `v'`
// from the vertex with the included angle.  The squared lengths are equal:
// |u|^2 = |u'|^2 and |v|^2 = |v'|^2.  The included angles are compared
// through their cosines: for equal side lengths the cosine of the included
// angle is `(u . v) / |u| |v|`, so the first angle is larger exactly when
// `u . v < u' . v'`.  The conclusion compares the squared third sides
// |u - v|^2 > |u' - v'|^2, which is the squared form of the hinge
// statement; expanding both sides with the squared-difference identity
// |x - y|^2 = |x|^2 + |y|^2 - 2 (x . y) reduces the claim to
// `-2 (u . v) > -2 (u' . v')`, which is the negation of `u . v < u' . v'`.

/// The hinge theorem: two triangles with two equal sides and a larger
/// included angle have a longer third side.
///
/// If the side vectors `u`, `v` and `u'`, `v'` satisfy |u|^2 = |u'|^2 and
/// |v|^2 = |v'|^2, and the included angle of the first triangle is larger
/// (`u . v < u' . v'`, since the cosine of a larger angle is smaller), then
/// the third side of the first triangle is longer:
/// |u - v|^2 > |u' - v'|^2.
theorem theorems1000_hinge(
    u: Pair[Real, Real], v: Pair[Real, Real],
    up: Pair[Real, Real], vp: Pair[Real, Real]
) {
    r2_norm_sq(u) = r2_norm_sq(up) and
    r2_norm_sq(v) = r2_norm_sq(vp) and
    r2_dot(u, v) < r2_dot(up, vp)
    implies
    r2_norm_sq(r2_sub(up, vp)) < r2_norm_sq(r2_sub(u, v))
} by {
    if r2_norm_sq(u) = r2_norm_sq(up) and
        r2_norm_sq(v) = r2_norm_sq(vp) and
        r2_dot(u, v) < r2_dot(up, vp) {
        r2_norm_sq(u) = r2_norm_sq(up)
        r2_norm_sq(v) = r2_norm_sq(vp)
        r2_dot(u, v) < r2_dot(up, vp)
        // expand both third-side squares
        r2_norm_sq_sub_expansion(u, v)
        r2_norm_sq(r2_sub(u, v)) = r2_norm_sq(u) + r2_norm_sq(v) - r2_dot(u, v) - r2_dot(u, v)
        r2_norm_sq_sub_expansion(up, vp)
        r2_norm_sq(r2_sub(up, vp)) = r2_norm_sq(up) + r2_norm_sq(vp) - r2_dot(up, vp) - r2_dot(up, vp)
        // the common base of the two expansions is equal
        r2_norm_sq(u) + r2_norm_sq(v) = r2_norm_sq(up) + r2_norm_sq(vp)
        // |x|^2 + |y|^2 - x.y - x.y = (|x|^2 + |y|^2) + (-x.y + -x.y)
        r2_norm_sq(u) + r2_norm_sq(v) - r2_dot(u, v) - r2_dot(u, v) =
            (r2_norm_sq(u) + r2_norm_sq(v)) + (-r2_dot(u, v) + -r2_dot(u, v))
        r2_norm_sq(up) + r2_norm_sq(vp) - r2_dot(up, vp) - r2_dot(up, vp) =
            (r2_norm_sq(up) + r2_norm_sq(vp)) + (-r2_dot(up, vp) + -r2_dot(up, vp))
        // negating reverses: -(u' . v') < -(u . v)
        lt_swap_neg(r2_dot(u, v), r2_dot(up, vp))
        -r2_dot(up, vp) < -r2_dot(u, v)
        // doubling keeps the inequality: -(u'.v') + -(u'.v') < -(u.v) + -(u.v)
        lt_add_right(-r2_dot(up, vp), -r2_dot(u, v), -r2_dot(up, vp))
        -r2_dot(up, vp) + -r2_dot(up, vp) < -r2_dot(u, v) + -r2_dot(up, vp)
        lt_add_left(-r2_dot(up, vp), -r2_dot(u, v), -r2_dot(u, v))
        -r2_dot(u, v) + -r2_dot(up, vp) < -r2_dot(u, v) + -r2_dot(u, v)
        lt_trans(-r2_dot(up, vp) + -r2_dot(up, vp), -r2_dot(u, v) + -r2_dot(up, vp), -r2_dot(u, v) + -r2_dot(u, v))
        -r2_dot(up, vp) + -r2_dot(up, vp) < -r2_dot(u, v) + -r2_dot(u, v)
        // add the common base |u|^2 + |v|^2 = |u'|^2 + |v'|^2 to both sides
        lt_add_left(-r2_dot(up, vp) + -r2_dot(up, vp), -r2_dot(u, v) + -r2_dot(u, v), r2_norm_sq(up) + r2_norm_sq(vp))
        (r2_norm_sq(up) + r2_norm_sq(vp)) + (-r2_dot(up, vp) + -r2_dot(up, vp)) < (r2_norm_sq(up) + r2_norm_sq(vp)) + (-r2_dot(u, v) + -r2_dot(u, v))
        (r2_norm_sq(up) + r2_norm_sq(vp)) + (-r2_dot(u, v) + -r2_dot(u, v)) =
            (r2_norm_sq(u) + r2_norm_sq(v)) + (-r2_dot(u, v) + -r2_dot(u, v))
        (r2_norm_sq(up) + r2_norm_sq(vp)) + (-r2_dot(up, vp) + -r2_dot(up, vp)) < (r2_norm_sq(u) + r2_norm_sq(v)) + (-r2_dot(u, v) + -r2_dot(u, v))
        // back to the subtracted form on each side
        r2_norm_sq(up) + r2_norm_sq(vp) - r2_dot(up, vp) - r2_dot(up, vp) < (r2_norm_sq(u) + r2_norm_sq(v)) + (-r2_dot(u, v) + -r2_dot(u, v))
        r2_norm_sq(up) + r2_norm_sq(vp) - r2_dot(up, vp) - r2_dot(up, vp) < r2_norm_sq(u) + r2_norm_sq(v) - r2_dot(u, v) - r2_dot(u, v)
        // replace each side by its expanded third side
        r2_norm_sq(r2_sub(up, vp)) < r2_norm_sq(u) + r2_norm_sq(v) - r2_dot(u, v) - r2_dot(u, v)
        r2_norm_sq(r2_sub(up, vp)) < r2_norm_sq(r2_sub(u, v))
    }
}
