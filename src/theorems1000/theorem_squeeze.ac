from nat import Nat
from real import Real, converges_to, tail_bound, tail_bound_implies_is_close,
    close_imp_bounds, bounds_imp_close, seq_lte
from order import lt_of_lt_of_lte, lte_lt_trans, lte_trans

numerals Real

// The squeeze theorem (also called the sandwich theorem or the pinching
// theorem): if a real sequence `b` is pointwise squeezed between two
// sequences `a` and `c`,
//     a(n) <= b(n) <= c(n)  for every n,
// and `a` and `c` both converge to the same limit `l`, then `b` converges
// to `l` as well.  The theorem is usually attributed to Carl Friedrich
// Gauss (who used it around 1810) and was stated explicitly by
// Augustin-Louis Cauchy; it is one of the oldest limit theorems and the
// standard tool for evaluating limits that cannot be computed directly,
// such as `x.sin/x -> 1`, `(1 + 1/n)^n -> e`, and the limit defining the
// number `e` itself.
//
// The formal statement uses the library's sequence notation: `seq_lte(a,
// b)` is the pointwise comparison `a(n) <= b(n)` for all `n`, and
// `converges_to(q, x)` is the Weierstrass definition of convergence.
//
// Proof (the classical epsilon argument): fix `eps > 0`.  Since `a` and `c`
// converge to `l`, there are indices `n_a` and `n_c` such that for every
// index past `n_a` the term `a(i)` is within `eps` of `l`, and likewise for
// `c` past `n_c`.  Past the sum `n_a + n_c` the chain
//     l - eps < a(i) <= b(i) <= c(i) < l + eps
// holds by the two squeezes and the closeness of the outer sequences, so
// `b(i)` is within `eps` of `l`.  Hence `b` satisfies the Weierstrass
// condition at `l`, which is the unfolding of `converges_to(b, l)`.

/// The squeeze theorem for real sequences: a sequence squeezed between two
/// sequences converging to the same limit converges to that limit.
theorem theorems1000_squeeze(a: Nat -> Real, b: Nat -> Real, c: Nat -> Real, l: Real) {
    seq_lte(a, b) and seq_lte(b, c) and converges_to(a, l) and converges_to(c, l)
    implies converges_to(b, l)
} by {
    if seq_lte(a, b) and seq_lte(b, c) and converges_to(a, l) and converges_to(c, l) {
        // The epsilon argument: for every positive eps there is a tail of b
        // within eps of l, squeezed between the tails of a and c.
        forall(eps: Real) {
            if eps.is_positive {
                // Unfold the Weierstrass definitions of the two outer
                // convergences and pick indices past which both hold.
                (converges_to(a, l) = forall(d: Real) {
                    d.is_positive implies exists(n: Nat) { tail_bound(a, l, n, d) }
                })
                exists(n: Nat) { tail_bound(a, l, n, eps) }
                let n_a: Nat satisfy { tail_bound(a, l, n_a, eps) }
                (converges_to(c, l) = forall(d: Real) {
                    d.is_positive implies exists(n: Nat) { tail_bound(c, l, n, d) }
                })
                exists(n: Nat) { tail_bound(c, l, n, eps) }
                let n_c: Nat satisfy { tail_bound(c, l, n_c, eps) }
                // From index n_a + n_c on, both bounds apply, so b is
                // squeezed between a and c and hence within eps of l.
                (tail_bound(b, l, n_a + n_c, eps) = forall(i: Nat) {
                    n_a + n_c <= i implies b(i).is_close(l, eps)
                })
                forall(i: Nat) {
                    if n_a + n_c <= i {
                        n_a <= n_a + n_c
                        lte_trans(n_a, n_a + n_c, i)
                        n_a <= i
                        tail_bound_implies_is_close(a, l, n_a, eps, i)
                        a(i).is_close(l, eps)
                        n_c <= n_a + n_c
                        lte_trans(n_c, n_a + n_c, i)
                        n_c <= i
                        tail_bound_implies_is_close(c, l, n_c, eps, i)
                        c(i).is_close(l, eps)
                        close_imp_bounds(a(i), l, eps)
                        l - eps < a(i) and a(i) < l + eps
                        l - eps < a(i)
                        seq_lte(a, b)
                        a(i) <= b(i)
                        lt_of_lt_of_lte(l - eps, a(i), b(i))
                        l - eps < b(i)
                        close_imp_bounds(c(i), l, eps)
                        c(i) > l - eps and c(i) < l + eps
                        c(i) < l + eps
                        seq_lte(b, c)
                        b(i) <= c(i)
                        lte_lt_trans(b(i), c(i), l + eps)
                        b(i) < l + eps
                        bounds_imp_close(b(i), l, eps)
                        b(i).is_close(l, eps)
                    }
                }
                tail_bound(b, l, n_a + n_c, eps)
                exists(n: Nat) { tail_bound(b, l, n, eps) }
            }
            (eps.is_positive implies exists(n: Nat) { tail_bound(b, l, n, eps) })
        }
    }
}
