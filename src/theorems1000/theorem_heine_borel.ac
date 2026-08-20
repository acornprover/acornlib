from real import Real
from order_set import closed_interval_set
from analysis import is_compact, heine_borel_closed_interval_is_compact

numerals Real

// The Heine–Borel theorem (Eduard Heine, 1872; Émile Borel, 1895): a
// subset of Euclidean space is compact exactly when it is closed and
// bounded.  On the real line the theorem takes the classical open-cover
// form: every family of open sets covering a closed bounded interval
// `[a, b]` has a finite subfamily that still covers it.  The theorem is
// the topological compactness property of the real numbers; together with
// Bolzano–Weierstrass it is one of the two standard formulations of the
// completeness of the reals.  It underlies the uniform continuity of
// continuous functions on closed intervals, the existence of extrema, the
// Lebesgue number lemma, and the convergence of the Riemann sums of
// continuous functions.
//
// The formal statement below is the open-cover form in the library's
// topology: `is_compact[Real](closed_interval_set(a, b))` says that every
// open cover of the closed interval `[a, b]` admits a finite subcover.
//
// The library proves the open-cover compactness of closed bounded
// intervals in `analysis/topology/real_topology.ac`
// (`heine_borel_closed_interval`) and restates it in
// `analysis/compactness_theorems.ac`
// (`heine_borel_closed_interval_is_compact`); the wrapper below restates
// that result with the ordered-endpoint hypothesis `a <= b` made
// explicit.

/// The Heine–Borel theorem: a closed bounded interval of the real line is
/// compact, i.e. every open cover of it admits a finite subcover.
theorem theorems1000_heine_borel(a: Real, b: Real) {
    a <= b implies is_compact[Real](closed_interval_set(a, b))
} by {
    if a <= b {
        heine_borel_closed_interval_is_compact(a, b)
        is_compact[Real](closed_interval_set(a, b))
    }
}
