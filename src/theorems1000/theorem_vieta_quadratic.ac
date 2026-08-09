from algebra.integral_domain import integral_domain_mul_eq_zero
from real import Real
from theorems1000.r2_helpers import real_sub_add_distrib, real_add_sub_cancel_middle, real_add_six_pair_rearrange, real_add_pair_rearrange

// Vieta's formulas for a quadratic polynomial: if the distinct real numbers
// `r1` and `r2` are the roots of `x^2 + b*x + c`, then
// `r1 + r2 = -b` and `r1 * r2 = c`.

/// The value of the quadratic `x^2 + b*x + c` at `x`.
define quadratic_value(b: Real, c: Real, x: Real) -> Real {
    x * x + b * x + c
}

/// A difference of squares factors into a product of a difference and a sum.
theorem real_square_diff(x: Real, y: Real) {
    x * x - y * y = (x - y) * (x + y)
} by {
    (x - y) * (x + y) = (x - y) * x + (x - y) * y
    (x - y) * x = x * x - y * x
    (x - y) * y = x * y - y * y
    (x - y) * (x + y) = x * x - y * x + (x * y - y * y)
    y * x = x * y
    (x - y) * (x + y) = x * x - x * y + (x * y - y * y)
    x * x - x * y + (x * y - y * y) = (x * x - x * y) + (x * y - y * y)
    real_sub_add_distrib(x * x, x * y, x * y, y * y)
    (x * x - x * y) + (x * y - y * y) = (x * x + x * y) - (x * y + y * y)
    real_add_sub_cancel_middle(x * x, y * y, x * y)
    (x * x + x * y) - (x * y + y * y) = x * x - y * y
    x * x - x * y + (x * y - y * y) = x * x - y * y
}

/// Subtracting a three-term sum termwise:
/// (a + b + c) - (d + e + f) = (a - d) + (b - e) + (c - f).
theorem real_sub_add_three(a: Real, b: Real, c: Real, d: Real, e: Real, f: Real) {
    a + b + c - (d + e + f) = (a - d) + (b - e) + (c - f)
} by {
    a + b + c - (d + e + f) = (a + b + c) + -(d + e + f)
    -(d + e + f) = -d + -e + -f
    (a + b + c) + -(d + e + f) = (a + b + c) + (-d + -e + -f)
    (a + b + c) + (-d + -e + -f) = a + b + c + -d + -e + -f
    real_add_six_pair_rearrange(a, b, c, -d, -e, -f)
    a + b + c + -d + -e + -f = (a + -d) + (c + -f) + (b + -e)
    (a + -d) + (c + -f) + (b + -e) = (a + -d) + (b + -e) + (c + -f)
    (a + -d) + (b + -e) + (c + -f) = (a - d) + (b - e) + (c - f)
}

/// The value at a point is the quadratic in expanded form.
theorem quadratic_value_expand(b: Real, c: Real, x: Real) {
    quadratic_value(b, c, x) = x * x + b * x + c
}

/// The sum of the roots of `x^2 + b*x + c` is `-b`.
theorem vieta_root_sum(r1: Real, r2: Real, b: Real, c: Real) {
    r1 != r2 and quadratic_value(b, c, r1) = Real.0 and quadratic_value(b, c, r2) = Real.0 implies
    r1 + r2 = -b
} by {
    if r1 != r2 and quadratic_value(b, c, r1) = Real.0 and quadratic_value(b, c, r2) = Real.0 {
        quadratic_value(b, c, r1) = r1 * r1 + b * r1 + c
        quadratic_value(b, c, r2) = r2 * r2 + b * r2 + c
        r1 * r1 + b * r1 + c = Real.0
        r2 * r2 + b * r2 + c = Real.0
        // subtracting the two root equations
        r1 * r1 + b * r1 + c - (r2 * r2 + b * r2 + c) = Real.0 - Real.0
        Real.0 - Real.0 = Real.0
        r1 * r1 + b * r1 + c - (r2 * r2 + b * r2 + c) = Real.0
        real_sub_add_three(r1 * r1, b * r1, c, r2 * r2, b * r2, c)
        r1 * r1 + b * r1 + c - (r2 * r2 + b * r2 + c) =
            (r1 * r1 - r2 * r2) + (b * r1 - b * r2) + (c - c)
        c - c = Real.0
        (r1 * r1 - r2 * r2) + (b * r1 - b * r2) + (c - c) =
            (r1 * r1 - r2 * r2) + (b * r1 - b * r2)
        (r1 * r1 - r2 * r2) + (b * r1 - b * r2) = Real.0
        // factoring
        real_square_diff(r1, r2)
        r1 * r1 - r2 * r2 = (r1 - r2) * (r1 + r2)
        b * r1 - b * r2 = b * (r1 - r2)
        (r1 - r2) * (r1 + r2) + b * (r1 - r2) = Real.0
        (r1 - r2) * (r1 + r2) + b * (r1 - r2) = (r1 - r2) * (r1 + r2 + b)
        (r1 - r2) * (r1 + r2 + b) = Real.0
        r1 - r2 != Real.0
        integral_domain_mul_eq_zero(r1 - r2, r1 + r2 + b)
        r1 - r2 = Real.0 or r1 + r2 + b = Real.0
        if r1 - r2 = Real.0 {
            false
        }
        r1 + r2 + b = Real.0
        (r1 + r2) + b = Real.0
        (r1 + r2) + b + -b = Real.0 + -b
        (r1 + r2) + (b + -b) = Real.0 + -b
        b + -b = Real.0
        (r1 + r2) + Real.0 = Real.0 + -b
        (r1 + r2) + Real.0 = r1 + r2
        Real.0 + -b = -b
        r1 + r2 = -b
    }
}

/// The product of the roots of `x^2 + b*x + c` is `c`.
theorem vieta_root_product(r1: Real, r2: Real, b: Real, c: Real) {
    r1 != r2 and quadratic_value(b, c, r1) = Real.0 and quadratic_value(b, c, r2) = Real.0 implies
    r1 * r2 = c
} by {
    if r1 != r2 and quadratic_value(b, c, r1) = Real.0 and quadratic_value(b, c, r2) = Real.0 {
        quadratic_value(b, c, r1) = r1 * r1 + b * r1 + c
        r1 * r1 + b * r1 + c = Real.0
        vieta_root_sum(r1, r2, b, c)
        r1 + r2 = -b
        r1 * r1 + b * r1 = r1 * (r1 + b)
        r1 * (r1 + b) + c = Real.0
        r1 * (r1 + b) + c + -c = Real.0 + -c
        r1 * (r1 + b) + (c + -c) = Real.0 + -c
        c + -c = Real.0
        r1 * (r1 + b) + Real.0 = Real.0 + -c
        r1 * (r1 + b) + Real.0 = r1 * (r1 + b)
        Real.0 + -c = -c
        r1 * (r1 + b) = -c
        r1 + r2 = -b
        r1 + r2 + b = -b + b
        -b + b = Real.0
        r1 + r2 + b = Real.0
        r1 + (r2 + b) = Real.0
        r2 + b = b + r2
        r1 + (b + r2) = Real.0
        r1 + (b + r2) + -(b + r2) = Real.0 + -(b + r2)
        r1 + (b + r2) + -(b + r2) = r1 + ((b + r2) + -(b + r2))
        (b + r2) + -(b + r2) = Real.0
        r1 + ((b + r2) + -(b + r2)) = r1 + Real.0
        r1 + Real.0 = r1
        Real.0 + -(b + r2) = -(b + r2)
        r1 = -(b + r2)
        -(b + r2) = -b + -r2
        r1 = -b + -r2
        r1 + b = -b + -r2 + b
        -b + -r2 + b = (-b + -r2) + b
        (-b + -r2) + b = (-b + -r2) + (b + Real.0)
        real_add_pair_rearrange(-b, -r2, b, Real.0)
        (-b + -r2) + (b + Real.0) = (-b + b) + (-r2 + Real.0)
        -b + b = Real.0
        -r2 + Real.0 = -r2
        (-b + b) + (-r2 + Real.0) = Real.0 + -r2
        Real.0 + -r2 = -r2
        r1 + b = -r2
        r1 * (r1 + b) = r1 * (-r2)
        r1 * (-r2) = -(r1 * r2)
        -c = -(r1 * r2)
        c = r1 * r2
    }
}

/// Vieta's formulas for a quadratic: the sum of the roots is `-b` and their
/// product is `c`.
theorem theorems1000_vieta_quadratic(r1: Real, r2: Real, b: Real, c: Real) {
    r1 != r2 and quadratic_value(b, c, r1) = Real.0 and quadratic_value(b, c, r2) = Real.0 implies
    r1 + r2 = -b and r1 * r2 = c
} by {
    if r1 != r2 and quadratic_value(b, c, r1) = Real.0 and quadratic_value(b, c, r2) = Real.0 {
        vieta_root_sum(r1, r2, b, c)
        r1 + r2 = -b
        vieta_root_product(r1, r2, b, c)
        r1 * r2 = c
        r1 + r2 = -b and r1 * r2 = c
    }
}
