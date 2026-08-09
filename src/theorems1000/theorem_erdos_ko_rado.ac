from nat import Nat
from list import List
from combinatorics import binom

numerals Nat

// The Erdős–Ko–Rado theorem (Paul Erdős, Chao Ko, Richard Rado, 1938;
// published 1961): fix natural numbers `n >= 2 k`.  A family `F` of
// `k`-element subsets of `{0, 1, ..., n - 1}` is *intersecting* when any two
// members share an element.  The theorem bounds the size of such a family:
//     |F| <= binom(n - 1, k - 1),
// and the bound is attained by the family of all `k`-subsets containing a
// fixed element (a "star").
//
// The theorem is the central result of extremal set theory.  The general
// version — the same bound for intersecting families of `k`-subsets of an
// arbitrary `n`-element set, for `n >= 2 k` — was conjectured by Erdős, Ko
// and Rado and proved by them for `n` large; the case `2 k <= n` was
// completed later (the "2 k <= n" range is exactly where the star is
// extremal).
//
// The formal statement below encodes the ground set as the naturals below
// `n`, a `k`-subset as a unique list of `k` elements below `n`, and the
// family `F` as a unique list of such lists.  Intersecting means that any
// two members share at least one element.
//
// Proof sketch (Katona's cycle method, 1972): arrange the `n` elements on a
// circle.  A family of `k`-subsets that are *intervals* of the cyclic order
// is intersecting only if the intervals all contain a common point, so at
// most `k` of the `n` cyclic intervals of length `k` can lie in the family.
// Counting each `k`-subset over all `n` cyclic orders (each subset appears
// in `k! (n - k)!` orders), an intersecting family of size `m` contributes
// at most `m k (n - k)!` intersecting interval systems, while the total
// number of interval systems across all orders is `n k! (n - k)!`; the two
// counts give `m <= n k! (n - k)! / (k (n - k)!) = binom(n - 1, k - 1)`.
// The library has finite-set cardinality machinery but not yet the cyclic
// counting, so the statement is recorded without proof.

// theorem theorems1000_erdos_ko_rado(n: Nat, k: Nat, F: List[List[Nat]]) {
//     n >= Nat.2 * k and F.is_unique and
//     (forall(S: List[Nat]) {
//         F.contains(S) implies
//             S.is_unique and S.length = k and
//             forall(x: Nat) { S.contains(x) implies x < n }
//     }) and
//     (forall(S: List[Nat], T: List[Nat]) {
//         F.contains(S) and F.contains(T) implies
//             exists(x: Nat) { S.contains(x) and T.contains(x) }
//     })
//         implies F.length <= (n - Nat.1).binom(k - Nat.1)
// }
