from pair import Pair
from real import Real
from top100 import r2_sub, r2_norm_sq
from theorems1000.r2_helpers import r2_cross

// Brahmagupta's theorem (Brahmagupta's formula): the area of a cyclic
// quadrilateral with side lengths `p`, `q`, `r`, `t` and semiperimeter
// `s = (p + q + r + t) / 2` is
//     K^2 = (s - p) (s - q) (s - r) (s - t),
// equivalently
//     16 K^2 = (p + q + r - t) (p + q - r + t) (p - q + r + t) (-p + q + r + t).
// It generalizes Heron's formula, which is the degenerate case `t = 0`.
//
// The statement is recorded with the quadrilateral `a b c d` on a circle of
// squared radius `radius_sq` centered at `o`, the side lengths given by the
// witnesses `p`, `q`, `r`, `t` with `p^2 = |b - a|^2`, `q^2 = |c - b|^2`,
// `r^2 = |d - c|^2`, `t^2 = |a - d|^2`, and the area witness `K` with
// `4 K = cross(b - a, c - a) + cross(c - a, d - a)` (twice the doubled area
// of the quadrilateral, the sum of the doubled areas of the triangles
// `abc` and `acd`, for a convex quadrilateral).
//
// Proof sketch: the cyclic condition makes opposite angles supplementary,
// so if `theta` is the angle between the diagonals of the two triangles
// `abc` and `acd`, `Real.sin^2(theta)` is the same for both and
// `16 K^2 = (p^2 + q^2 + r^2 + t^2)^2 - 8 p q r t - 2(p^4 + q^4 + r^4 + t^4)`
// after applying the law of cosines to express the diagonal `ac` twice and
// eliminating it; the right-hand side factors as
// `(p+q+r-t)(p+q-r+t)(p-q+r+t)(-p+q+r+t)`.  The factorization is a
// polynomial identity, and the diagonal elimination is the same algebra as
// in Heron's formula (`top100_057_herons_formula_squared` in this library).
//
// theorem theorems1000_brahmagupta(
//     o: Pair[Real, Real], radius_sq: Real,
//     a: Pair[Real, Real], b: Pair[Real, Real],
//     c: Pair[Real, Real], d: Pair[Real, Real],
//     p: Real, q: Real, r: Real, t: Real, k: Real
// ) {
//     r2_norm_sq(r2_sub(a, o)) = radius_sq and
//     r2_norm_sq(r2_sub(b, o)) = radius_sq and
//     r2_norm_sq(r2_sub(c, o)) = radius_sq and
//     r2_norm_sq(r2_sub(d, o)) = radius_sq and
//     p * p = r2_norm_sq(r2_sub(b, a)) and
//     q * q = r2_norm_sq(r2_sub(c, b)) and
//     r * r = r2_norm_sq(r2_sub(d, c)) and
//     t * t = r2_norm_sq(r2_sub(a, d)) and
//     (Real.1 + Real.1 + Real.1 + Real.1) * k =
//         r2_cross(r2_sub(b, a), r2_sub(c, a)) + r2_cross(r2_sub(c, a), r2_sub(d, a))
//     implies
//     (Real.1 + Real.1 + Real.1 + Real.1 + Real.1 + Real.1 + Real.1 + Real.1 +
//         Real.1 + Real.1 + Real.1 + Real.1 + Real.1 + Real.1 + Real.1 + Real.1) *
//         (k * k) =
//         (p + q + r - t) * (p + q - r + t) * (p - q + r + t) * (-p + q + r + t)
// }
