from pair import Pair
from real import Real, real_mul_comm, mul_assoc
from top100 import r2_add, r2_neg, r2_sub, r2_dot, r2_norm_sq, r2_orthogonal
from theorems1000.r2_helpers import r2_smul, r2_sub_through, r2_sub_eq_add_neg,
    r2_sub_reverse_neg, r2_norm_sq_add_expansion, r2_norm_sq_smul, r2_dot_smul_right,
    r2_pythagoras_vectors, r2_norm_sq_neg, r2_add_sub_left_cancel, r2_dot_comm
from algebra.field.field import field_mul_eq_zero
from algebra.add_comm_group import sub_eq_zero_imp_eq
from algebra.add_group import right_cancel

numerals Real

// The tangent-secant theorem (the power of a point): from a point `p`
// outside a circle, if a tangent touches the circle at `a` and a secant
// through `p` meets the circle at `b` and `c`, then the squared tangent
// length equals the product of the two secant segments:
//     |p-a|^2 = |p-b| * |p-c|   (as lengths),
// equivalently, in squared form,
//     |p-a|^2 * |p-a|^2 = |p-b|^2 * |p-c|^2.
// This is the limiting case of the intersecting-secants theorem (the two
// secant intersection points of a secant that has become a tangent
// coincide), and together with the intersecting-chords theorem it expresses
// the constancy of the power of a point with respect to a circle.
//
// The statement uses the library's witness convention: the circle has center
// `o` and squared radius `radius_sq`; the tangent point `a` satisfies
// `|a-o|^2 = radius_sq` and the tangency condition `(p - a) . (a - o) = 0`
// (the tangent is perpendicular to the radius); the secant is the line
// `p + s w` through `p` with direction `w`, meeting the circle at
// `b = p + s w` and `c = p + t w`.
//
// Both sides equal the square of the power of `p`, `(|p-o|^2 - radius_sq)^2`:
// for the tangent, Pythagoras in the right triangle `p o a`; for the secant,
// the two circle memberships expand to one quadratic in the parameter whose
// two roots are `s` and `t`, and the root product is the power.  The library
// proves the same statement as `point2_tangent_secant_power` in
// `geometry/circle_theorems.ac` (private to the `geometry` package); the
// proof below develops it from the `r2` vector algebra of this directory.

/// Moving a term across a quadratic equation:
/// `x + y + z = 0` gives `x + z = -y`.
theorem ts_quadratic_move_term(x: Real, y: Real, z: Real) {
    x + y + z = Real.0 implies x + z = -y
} by {
    if x + y + z = Real.0 {
        x + y + z = Real.0
        (x + z) + y = x + y + z
        (x + z) + y = Real.0
        -y + y = Real.0
        (x + z) + y = -y + y
        right_cancel(x + z, -y, y)
        x + z = -y
    }
}

/// Subtracting the right-hand side of an equality gives zero.
theorem ts_equality_sub_rhs_zero(x: Real, y: Real) {
    x = y implies x - y = Real.0
} by {
    if x = y {
        x - y = Real.0
    }
}

/// The difference of two two-term sums factors into a difference of
/// products: `(u s + c t) - (u t + c s) = u (s - t) - c (s - t)`.
theorem ts_difference_factor(u: Real, c: Real, s: Real, t: Real) {
    (u * s + c * t) - (u * t + c * s) = u * (s - t) - c * (s - t)
} by {
    (u * s + c * t) - (u * t + c * s) = (u * s - u * t) + (c * t - c * s)
    u * (s - t) = u * s - u * t
    c * (s - t) = c * s - c * t
    (u * s - u * t) + (c * t - c * s) = u * (s - t) - c * (s - t)
}

/// An equality of two two-term sums factors into an equality of products.
theorem ts_factor_move(u: Real, c: Real, s: Real, t: Real) {
    u * s + c * t = u * t + c * s implies
    u * (s - t) = c * (s - t)
} by {
    if u * s + c * t = u * t + c * s {
        u * s + c * t = u * t + c * s
        ts_equality_sub_rhs_zero(u * s + c * t, u * t + c * s)
        (u * s + c * t) - (u * t + c * s) = Real.0
        ts_difference_factor(u, c, s, t)
        (u * s + c * t) - (u * t + c * s) = u * (s - t) - c * (s - t)
        u * (s - t) - c * (s - t) = Real.0
        u * (s - t) = c * (s - t)
    }
}

/// An equality of factored products is a zero product.
theorem ts_equality_to_zero_product(u: Real, c: Real, s: Real, t: Real) {
    u * (s - t) = c * (s - t) implies
    (s - t) * (u - c) = Real.0
} by {
    if u * (s - t) = c * (s - t) {
        u * (s - t) = c * (s - t)
        u * (s - t) - c * (s - t) = Real.0
        (s - t) * (u - c) = (s - t) * u - (s - t) * c
        (s - t) * u = u * (s - t)
        (s - t) * c = c * (s - t)
        (s - t) * (u - c) = u * (s - t) - c * (s - t)
        (s - t) * (u - c) = Real.0
    }
}

/// The product of the two roots of a quadratic: if `a s^2 + b s + c = 0`
/// and `a t^2 + b t + c = 0` with `s != t`, then `a s t = c`.
theorem ts_root_product(a: Real, b: Real, c: Real, s: Real, t: Real) {
    a * s * s + b * s + c = Real.0 and
    a * t * t + b * t + c = Real.0 and
    s != t
    implies a * s * t = c
} by {
    if a * s * s + b * s + c = Real.0 and
        a * t * t + b * t + c = Real.0 and
        s != t {
        a * s * s + b * s + c = Real.0
        a * t * t + b * t + c = Real.0
        s != t
        if s - t = Real.0 {
            sub_eq_zero_imp_eq(s, t)
            s = t
            false
        }
        s - t != Real.0
        (a * s * s + b * s + c) * t = Real.0
        (a * s * s + b * s + c) * t = a * s * s * t + b * s * t + c * t
        a * s * s * t + b * s * t + c * t = Real.0
        ts_quadratic_move_term(a * s * s * t, b * s * t, c * t)
        a * s * s * t + c * t = -(b * s * t)
        (a * t * t + b * t + c) * s = Real.0
        (a * t * t + b * t + c) * s = a * t * t * s + b * t * s + c * s
        a * t * t * s + b * t * s + c * s = Real.0
        ts_quadratic_move_term(a * t * t * s, b * t * s, c * s)
        a * t * t * s + c * s = -(b * t * s)
        real_mul_comm(s, t)
        s * t = t * s
        b * (s * t) = b * (t * s)
        mul_assoc(b, s, t)
        b * s * t = b * (s * t)
        b * (t * s) = b * t * s
        b * s * t = b * t * s
        -(b * s * t) = -(b * t * s)
        a * t * t * s + c * s = -(b * s * t)
        a * s * s * t + c * t = a * t * t * s + c * s
        real_mul_comm(a, s)
        a * s = s * a
        mul_assoc(a * s, t, s)
        (a * s * t) * s = (a * s) * (t * s)
        real_mul_comm(t, s)
        t * s = s * t
        (a * s) * (t * s) = (a * s) * (s * t)
        mul_assoc(a, s, s * t)
        a * (s * (s * t)) = a * s * (s * t)
        mul_assoc(a * s, s, t)
        (a * s) * (s * t) = a * s * (s * t)
        mul_assoc(a, s, s * t)
        a * (s * (s * t)) = (a * s) * (s * t)
        mul_assoc(s, s, t)
        s * (s * t) = s * s * t
        a * (s * s * t) = a * s * s * t
        mul_assoc(a * s, s, t)
        a * s * (s * t) = a * s * s * t
        (a * s * t) * s = a * s * s * t
        mul_assoc(a * s, t, t)
        (a * s * t) * t = (a * s) * (t * t)
        real_mul_comm(s, t)
        s * t = t * s
        mul_assoc(a, s, t * t)
        a * (s * (t * t)) = a * s * (t * t)
        real_mul_comm(s, t * t)
        s * (t * t) = (t * t) * s
        a * ((t * t) * s) = a * s * (t * t)
        mul_assoc(t, t, s)
        (t * t) * s = t * (t * s)
        a * (t * (t * s)) = a * t * (t * s)
        mul_assoc(a * t, t, s)
        a * t * (t * s) = a * t * t * s
        mul_assoc(a, t, t * s)
        a * (t * (t * s)) = a * (t * (t * s))
        (a * s * t) * t = a * t * t * s
        (a * s * t) * s + c * t = (a * s * t) * t + c * s
        ts_factor_move(a * s * t, c, s, t)
        (a * s * t) * (s - t) = c * (s - t)
        ts_equality_to_zero_product(a * s * t, c, s, t)
        (s - t) * (a * s * t - c) = Real.0
        field_mul_eq_zero(s - t, a * s * t - c)
        s - t = Real.0 or a * s * t - c = Real.0
        if s - t = Real.0 {
            false
        } else {
            a * s * t - c = Real.0
            sub_eq_zero_imp_eq(a * s * t, c)
            a * s * t = c
        }
    }
}

/// The product of the two squared secant segments in terms of the parameter
/// product: `(s^2 W) (t^2 W) = (W s t)^2`.
theorem ts_secant_product_square(w_sq: Real, s: Real, t: Real) {
    (s * s * w_sq) * (t * t * w_sq) = (w_sq * s * t) * (w_sq * s * t)
} by {
    // left: (s s w)(t t w) -> ((s s)(t t))(w w)
    mul_assoc(s * s, w_sq, t * t * w_sq)
    (s * s * w_sq) * (t * t * w_sq) = (s * s) * (w_sq * (t * t * w_sq))
    real_mul_comm(w_sq, t * t * w_sq)
    w_sq * (t * t * w_sq) = (t * t * w_sq) * w_sq
    (s * s) * (w_sq * (t * t * w_sq)) = (s * s) * ((t * t * w_sq) * w_sq)
    mul_assoc(s * s, t * t * w_sq, w_sq)
    (s * s) * ((t * t * w_sq) * w_sq) = (s * s) * (t * t * w_sq) * w_sq
    mul_assoc(s * s, t * t, w_sq)
    (s * s) * ((t * t) * w_sq) = ((s * s) * (t * t)) * w_sq
    (s * s) * (t * t * w_sq) = ((s * s) * (t * t)) * w_sq
    (s * s) * (t * t * w_sq) * w_sq = ((s * s) * (t * t)) * w_sq * w_sq
    mul_assoc((s * s) * (t * t), w_sq, w_sq)
    ((s * s) * (t * t)) * w_sq * w_sq = ((s * s) * (t * t)) * (w_sq * w_sq)
    (s * s * w_sq) * (t * t * w_sq) = ((s * s) * (t * t)) * (w_sq * w_sq)
    // right: (w s t)(w s t) -> ((w s)(w s))(t t)
    mul_assoc(w_sq * s, t, (w_sq * s) * t)
    (w_sq * s * t) * ((w_sq * s) * t) = (w_sq * s) * (t * ((w_sq * s) * t))
    real_mul_comm(t, (w_sq * s) * t)
    t * ((w_sq * s) * t) = ((w_sq * s) * t) * t
    (w_sq * s) * (t * ((w_sq * s) * t)) = (w_sq * s) * (((w_sq * s) * t) * t)
    mul_assoc(w_sq * s, (w_sq * s) * t, t)
    (w_sq * s) * (((w_sq * s) * t) * t) = (w_sq * s) * ((w_sq * s) * t) * t
    mul_assoc(w_sq * s, w_sq * s, t)
    (w_sq * s) * ((w_sq * s) * t) = ((w_sq * s) * (w_sq * s)) * t
    (w_sq * s) * ((w_sq * s) * t) * t = ((w_sq * s) * (w_sq * s)) * t * t
    mul_assoc((w_sq * s) * (w_sq * s), t, t)
    ((w_sq * s) * (w_sq * s)) * t * t = ((w_sq * s) * (w_sq * s)) * (t * t)
    (w_sq * s) * (w_sq * s) = w_sq * w_sq * s * s
    ((w_sq * s) * (w_sq * s)) * (t * t) = (w_sq * w_sq * s * s) * (t * t)
    // both sides are (w w)(s s)(t t) in different groupings
    w_sq * w_sq * s * s * (t * t) = (w_sq * w_sq) * (s * s) * (t * t)
    ((s * s) * (t * t)) * (w_sq * w_sq) = (w_sq * w_sq) * (s * s) * (t * t)
    (s * s * w_sq) * (t * t * w_sq) = (w_sq * s * t) * (w_sq * s * t)
}

/// Expanding a circle membership into a quadratic in the parameter:
/// `u + s^2 a + s d + s d = r` rearranges to `a s^2 + (d + d) s + (u - r) = 0`.
theorem ts_quadratic_rearrange(u: Real, a: Real, d: Real, s: Real, r: Real) {
    u + s * s * a + s * d + s * d = r implies
    a * s * s + (d + d) * s + (u - r) = Real.0
} by {
    if u + s * s * a + s * d + s * d = r {
        u + s * s * a + s * d + s * d = r
        ts_equality_sub_rhs_zero(u + s * s * a + s * d + s * d, r)
        u + s * s * a + s * d + s * d - r = Real.0
        (u + s * s * a + s * d + s * d) - r = a * s * s + (d + d) * s + (u - r)
        a * s * s + (d + d) * s + (u - r) = Real.0
    }
}

/// The displacement `p - o` through the tangent point: `p - o =
/// (p - a) + (a - o)`.
theorem ts_sub_through_tangent(a: Pair[Real, Real], o: Pair[Real, Real], p: Pair[Real, Real]) {
    r2_sub(p, o) = r2_add(r2_sub(p, a), r2_sub(a, o))
} by {
    r2_sub_through(a, o, p)
    r2_sub(p, a) = r2_add(r2_sub(o, a), r2_sub(p, o))
    r2_sub_reverse_neg(o, a)
    r2_sub(o, a) = r2_neg(r2_sub(a, o))
    r2_sub(p, a) = r2_add(r2_neg(r2_sub(a, o)), r2_sub(p, o))
    r2_sub_eq_add_neg(r2_sub(p, o), r2_sub(a, o))
    r2_sub(r2_sub(p, o), r2_sub(a, o)) =
        r2_add(r2_sub(p, o), r2_neg(r2_sub(a, o)))
    r2_sub(p, a) = r2_sub(r2_sub(p, o), r2_sub(a, o))
    r2_sub(p, o) = r2_add(r2_sub(p, a), r2_sub(a, o))
}

/// The squared tangent length is the power of the point:
/// `|p-a|^2 = |p-o|^2 - radius_sq` for a tangent at `a`.
theorem ts_tangent_power(
    o: Pair[Real, Real], radius_sq: Real,
    p: Pair[Real, Real], a: Pair[Real, Real]
) {
    r2_norm_sq(r2_sub(a, o)) = radius_sq and
    r2_dot(r2_sub(p, a), r2_sub(a, o)) = Real.0
    implies r2_norm_sq(r2_sub(p, a)) = r2_norm_sq(r2_sub(p, o)) - radius_sq
} by {
    if r2_norm_sq(r2_sub(a, o)) = radius_sq and
        r2_dot(r2_sub(p, a), r2_sub(a, o)) = Real.0 {
        // the right triangle p-o-a is right at a
        r2_orthogonal(r2_sub(p, a), r2_sub(a, o))
        r2_pythagoras_vectors(r2_sub(p, a), r2_sub(a, o))
        r2_norm_sq(r2_add(r2_sub(p, a), r2_sub(a, o))) =
            r2_norm_sq(r2_sub(p, a)) + r2_norm_sq(r2_sub(a, o))
        ts_sub_through_tangent(a, o, p)
        r2_sub(p, o) = r2_add(r2_sub(p, a), r2_sub(a, o))
        r2_norm_sq(r2_sub(p, o)) =
            r2_norm_sq(r2_sub(p, a)) + r2_norm_sq(r2_sub(a, o))
        r2_norm_sq(r2_sub(a, o)) = radius_sq
        r2_norm_sq(r2_sub(p, o)) = r2_norm_sq(r2_sub(p, a)) + radius_sq
        r2_norm_sq(r2_sub(p, a)) = r2_norm_sq(r2_sub(p, o)) - radius_sq
    }
}

/// The squared secant segment from the parameter: `b = p + s w` gives
/// `|p-b|^2 = s^2 |w|^2`.
theorem ts_secant_segment_sq(
    p: Pair[Real, Real], b: Pair[Real, Real], w: Pair[Real, Real], s: Real
) {
    r2_add(p, r2_smul(s, w)) = b
    implies r2_norm_sq(r2_sub(p, b)) = s * s * r2_norm_sq(w)
} by {
    if r2_add(p, r2_smul(s, w)) = b {
        r2_add_sub_left_cancel(p, r2_smul(s, w))
        r2_sub(r2_add(p, r2_smul(s, w)), p) = r2_smul(s, w)
        r2_sub(b, p) = r2_smul(s, w)
        r2_norm_sq(r2_sub(b, p)) = r2_norm_sq(r2_smul(s, w))
        r2_norm_sq_smul(s, w)
        r2_norm_sq(r2_smul(s, w)) = s * s * r2_norm_sq(w)
        r2_norm_sq(r2_sub(b, p)) = s * s * r2_norm_sq(w)
        r2_sub_reverse_neg(b, p)
        r2_sub(b, p) = r2_neg(r2_sub(p, b))
        r2_norm_sq_neg(r2_sub(p, b))
        r2_norm_sq(r2_neg(r2_sub(p, b))) = r2_norm_sq(r2_sub(p, b))
        r2_norm_sq(r2_sub(p, b)) = s * s * r2_norm_sq(w)
    }
}

/// The tangent-secant theorem (the power of a point): the squared tangent
/// length equals the product of the two squared secant segments.
theorem theorems1000_tangent_secant(
    o: Pair[Real, Real], radius_sq: Real,
    p: Pair[Real, Real], a: Pair[Real, Real],
    b: Pair[Real, Real], c: Pair[Real, Real],
    w: Pair[Real, Real], s: Real, t: Real
) {
    r2_norm_sq(r2_sub(a, o)) = radius_sq and
    r2_dot(r2_sub(p, a), r2_sub(a, o)) = Real.0 and
    r2_add(p, r2_smul(s, w)) = b and
    r2_add(p, r2_smul(t, w)) = c and
    r2_norm_sq(r2_sub(b, o)) = radius_sq and
    r2_norm_sq(r2_sub(c, o)) = radius_sq and
    s != t
    implies
    r2_norm_sq(r2_sub(p, a)) * r2_norm_sq(r2_sub(p, a)) =
        r2_norm_sq(r2_sub(p, b)) * r2_norm_sq(r2_sub(p, c))
} by {
    if r2_norm_sq(r2_sub(a, o)) = radius_sq and
        r2_dot(r2_sub(p, a), r2_sub(a, o)) = Real.0 and
        r2_add(p, r2_smul(s, w)) = b and
        r2_add(p, r2_smul(t, w)) = c and
        r2_norm_sq(r2_sub(b, o)) = radius_sq and
        r2_norm_sq(r2_sub(c, o)) = radius_sq and
        s != t {
        // the tangent: |p-a|^2 = |p-o|^2 - radius_sq
        ts_tangent_power(o, radius_sq, p, a)
        r2_norm_sq(r2_sub(p, a)) = r2_norm_sq(r2_sub(p, o)) - radius_sq
        // the secant: the circle membership of b = p + s w expands to a
        // quadratic in s
        r2_sub_through(o, p, b)
        r2_sub(b, o) = r2_add(r2_sub(p, o), r2_sub(b, p))
        ts_secant_segment_sq(p, b, w, s)
        r2_sub(b, p) = r2_smul(s, w)
        r2_sub(b, o) = r2_add(r2_sub(p, o), r2_smul(s, w))
        r2_norm_sq_add_expansion(r2_sub(p, o), r2_smul(s, w))
        r2_norm_sq(r2_add(r2_sub(p, o), r2_smul(s, w))) =
            r2_norm_sq(r2_sub(p, o)) + r2_norm_sq(r2_smul(s, w)) +
            r2_dot(r2_sub(p, o), r2_smul(s, w)) + r2_dot(r2_sub(p, o), r2_smul(s, w))
        r2_norm_sq_smul(s, w)
        r2_norm_sq(r2_smul(s, w)) = s * s * r2_norm_sq(w)
        r2_dot_smul_right(s, r2_sub(p, o), w)
        r2_dot(r2_sub(p, o), r2_smul(s, w)) = s * r2_dot(r2_sub(p, o), w)
        r2_norm_sq(r2_sub(b, o)) =
            r2_norm_sq(r2_sub(p, o)) + s * s * r2_norm_sq(w) +
            s * r2_dot(r2_sub(p, o), w) + s * r2_dot(r2_sub(p, o), w)
        r2_norm_sq(r2_sub(b, o)) = radius_sq
        r2_norm_sq(r2_sub(p, o)) + s * s * r2_norm_sq(w) +
            s * r2_dot(r2_sub(p, o), w) + s * r2_dot(r2_sub(p, o), w) = radius_sq
        // rearrange to a quadratic in s
        ts_quadratic_rearrange(r2_norm_sq(r2_sub(p, o)), r2_norm_sq(w),
            r2_dot(r2_sub(p, o), w), s, radius_sq)
        r2_norm_sq(w) * s * s +
            (r2_dot(r2_sub(p, o), w) + r2_dot(r2_sub(p, o), w)) * s +
            (r2_norm_sq(r2_sub(p, o)) - radius_sq) = Real.0
        // the same for t
        r2_sub_through(o, p, c)
        r2_sub(c, o) = r2_add(r2_sub(p, o), r2_sub(c, p))
        ts_secant_segment_sq(p, c, w, t)
        r2_sub(c, p) = r2_smul(t, w)
        r2_sub(c, o) = r2_add(r2_sub(p, o), r2_smul(t, w))
        r2_norm_sq_add_expansion(r2_sub(p, o), r2_smul(t, w))
        r2_norm_sq(r2_add(r2_sub(p, o), r2_smul(t, w))) =
            r2_norm_sq(r2_sub(p, o)) + r2_norm_sq(r2_smul(t, w)) +
            r2_dot(r2_sub(p, o), r2_smul(t, w)) + r2_dot(r2_sub(p, o), r2_smul(t, w))
        r2_norm_sq_smul(t, w)
        r2_norm_sq(r2_smul(t, w)) = t * t * r2_norm_sq(w)
        r2_dot_smul_right(t, r2_sub(p, o), w)
        r2_dot(r2_sub(p, o), r2_smul(t, w)) = t * r2_dot(r2_sub(p, o), w)
        r2_norm_sq(r2_sub(c, o)) =
            r2_norm_sq(r2_sub(p, o)) + t * t * r2_norm_sq(w) +
            t * r2_dot(r2_sub(p, o), w) + t * r2_dot(r2_sub(p, o), w)
        r2_norm_sq(r2_sub(c, o)) = radius_sq
        r2_norm_sq(r2_sub(p, o)) + t * t * r2_norm_sq(w) +
            t * r2_dot(r2_sub(p, o), w) + t * r2_dot(r2_sub(p, o), w) = radius_sq
        ts_quadratic_rearrange(r2_norm_sq(r2_sub(p, o)), r2_norm_sq(w),
            r2_dot(r2_sub(p, o), w), t, radius_sq)
        r2_norm_sq(w) * t * t +
            (r2_dot(r2_sub(p, o), w) + r2_dot(r2_sub(p, o), w)) * t +
            (r2_norm_sq(r2_sub(p, o)) - radius_sq) = Real.0
        // the product of the parameters is the power
        ts_root_product(r2_norm_sq(w),
            r2_dot(r2_sub(p, o), w) + r2_dot(r2_sub(p, o), w),
            r2_norm_sq(r2_sub(p, o)) - radius_sq, s, t)
        r2_norm_sq(w) * s * t = r2_norm_sq(r2_sub(p, o)) - radius_sq
        // |p-b|^2 * |p-c|^2 = (s^2 |w|^2)(t^2 |w|^2) = (|w|^2 s t)^2
        ts_secant_segment_sq(p, b, w, s)
        r2_norm_sq(r2_sub(p, b)) = s * s * r2_norm_sq(w)
        ts_secant_segment_sq(p, c, w, t)
        r2_norm_sq(r2_sub(p, c)) = t * t * r2_norm_sq(w)
        r2_norm_sq(r2_sub(p, b)) * r2_norm_sq(r2_sub(p, c)) =
            (s * s * r2_norm_sq(w)) * (t * t * r2_norm_sq(w))
        ts_secant_product_square(r2_norm_sq(w), s, t)
        (s * s * r2_norm_sq(w)) * (t * t * r2_norm_sq(w)) =
            (r2_norm_sq(w) * s * t) * (r2_norm_sq(w) * s * t)
        (r2_norm_sq(w) * s * t) * (r2_norm_sq(w) * s * t) =
            (r2_norm_sq(r2_sub(p, o)) - radius_sq) * (r2_norm_sq(r2_sub(p, o)) - radius_sq)
        r2_norm_sq(r2_sub(p, b)) * r2_norm_sq(r2_sub(p, c)) =
            (r2_norm_sq(r2_sub(p, o)) - radius_sq) * (r2_norm_sq(r2_sub(p, o)) - radius_sq)
        // the tangent side squares to the same value
        r2_norm_sq(r2_sub(p, a)) * r2_norm_sq(r2_sub(p, a)) =
            (r2_norm_sq(r2_sub(p, o)) - radius_sq) * (r2_norm_sq(r2_sub(p, o)) - radius_sq)
        r2_norm_sq(r2_sub(p, a)) * r2_norm_sq(r2_sub(p, a)) =
            r2_norm_sq(r2_sub(p, b)) * r2_norm_sq(r2_sub(p, c))
    }
}
