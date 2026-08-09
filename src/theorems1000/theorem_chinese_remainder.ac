from nat import Nat, mul_assoc, mul_comm, mul_one_right, mul_one_left, add_zero_right,
    add_zero_left, add_comm, divides_self, divides_mul, div_imp_mod, mod_of_zero, gcd_comm
from number_theory.interface import congr_mod_add, congr_mod_mul, congr_mod_refl,
    congr_mod_symm, congr_mod_trans, mod_inv, nat_congr_combine_coprime

numerals Nat

// The Chinese remainder theorem (Sunzi, c. 3rd century, in the
// "Sunzi suanjing"; also known from the work of Qin Jiushao and, in
// Europe, of Carl Friedrich Gauss in the Disquisitiones Arithmeticae,
// 1801): for coprime moduli `m` and `n`, every pair of residue
// requirements
//     c ≡ a  (mod m),     c ≡ b  (mod n)
// has a simultaneous solution `c`, and any two solutions are congruent
// modulo the product `m n`.  Historically the theorem was stated with
// concrete numbers ("a number leaves remainder 3 on division by 5, 2 by 7,
// and 3 by 11; what is it?"); in modern form it is the statement that the
// ring Z/(mn) is the product of the rings Z/(m) x Z/(n), and it underlies
// the multiplicative structure of Euler's totient function, the
// construction of solutions to systems of linear congruences, and the
// reconstruction of integers from their remainders (as used in
// secret-sharing and in the number-theoretic transform).
//
// The formal statement is in the library's congruence notation
// (`number_theory/congruence.ac`): `c.congr_mod(a, m)` is the congruence
// `c ≡ a (mod m)`, defined as the equality of the remainders
// `c.mod(m) = a.mod(m)`, and `m.coprime(n)` is the coprimality of the two
// moduli (their gcd is one).  The theorem below is the existence half; the
// uniqueness half follows from the library's `nat_congr_combine_coprime`
// and is stated as the second theorem of this file.
//
// Proof of existence (the classical construction): since `m` and `n` are
// coprime, `m` has a multiplicative inverse `u` modulo `n` and `n` has an
// inverse `v` modulo `m`, i.e.
//     m u ≡ 1 (mod n),     n v ≡ 1 (mod m)
// (the library's `mod_inv` in `number_theory/modular_inverse.ac` provides
// both inverses).  The number
//     c = a n v + b m u
// then satisfies both congruences:
//     mod m:  c ≡ a n v ≡ a (n v) ≡ a · 1 ≡ a   (mod m),
//     mod n:  c ≡ b m u ≡ b (m u) ≡ b · 1 ≡ b   (mod n),
// since the crossed terms `b m u` and `a n v` are multiples of `m` and `n`
// respectively and hence vanish modulo those moduli.  The proof below
// assembles these steps from the congruence calculus (`congr_mod_add`,
// `congr_mod_mul`) and the modular-inverse specification.

/// Coprimality is symmetric: `m.coprime(n)` implies `n.coprime(m)`.
theorem theorems1000_coprime_symm(m: Nat, n: Nat) {
    m.coprime(n) implies n.coprime(m)
} by {
    if m.coprime(n) {
        m.coprime(n) = (m.gcd(n) = Nat.1)
        m.gcd(n) = Nat.1
        gcd_comm(m, n)
        m.gcd(n) = n.gcd(m)
        n.gcd(m) = Nat.1
        n.coprime(m) = (n.gcd(m) = Nat.1)
        n.coprime(m)
    }
}

/// The modular inverse of a coprime modulus satisfies its defining
/// congruence: `m.coprime(n)` implies `(m * mod_inv(m, n)) ≡ 1 (mod n)`.
theorem theorems1000_mod_inv_congr_one(m: Nat, n: Nat) {
    m.coprime(n) implies (m * mod_inv(m, n)).congr_mod(Nat.1, n)
} by {
    if m.coprime(n) {
        (m * mod_inv(m, n)).congr_mod(Nat.1, n)
    }
}

/// Any multiple of the modulus is congruent to zero modulo it:
/// `(b * (m * u)) ≡ 0 (mod m)`.
theorem theorems1000_mul_congr_zero(b: Nat, m: Nat, u: Nat) {
    (b * (m * u)).congr_mod(Nat.0, m)
} by {
    // (b * (m * u)) = (b * u) * m is a multiple of m.
    b * (m * u) = (b * u) * m
    divides_self(m)
    m.divides(m)
    divides_mul(m, b * u, m)
    m.divides(m * (b * u))
    m * (b * u) = (b * u) * m
    m.divides((b * u) * m)
    (b * u) * m = b * (m * u)
    m.divides(b * (m * u))
    div_imp_mod(b * (m * u), m)
    (b * (m * u)).mod(m) = Nat.0
    mod_of_zero(m)
    Nat.0.mod(m) = Nat.0
    (b * (m * u)).mod(m) = Nat.0.mod(m)
    (b * (m * u)).congr_mod(Nat.0, m)
}

/// The Chinese remainder theorem, existence: for coprime moduli `m` and
/// `n`, every pair of residues `(a, b)` is realized by some `c`.
theorem theorems1000_chinese_remainder(m: Nat, n: Nat, a: Nat, b: Nat) {
    m.coprime(n) and m != Nat.0 and n != Nat.0
    implies exists(c: Nat) { c.congr_mod(a, m) and c.congr_mod(b, n) }
} by {
    if m.coprime(n) and m != Nat.0 and n != Nat.0 {
        // The witness is c = a n v + b m u with u = mod_inv(m, n) and
        // v = mod_inv(n, m).  First the two inverse congruences.
        theorems1000_mod_inv_congr_one(m, n)
        (m * mod_inv(m, n)).congr_mod(Nat.1, n)
        theorems1000_coprime_symm(m, n)
        n.coprime(m)
        theorems1000_mod_inv_congr_one(n, m)
        (n * mod_inv(n, m)).congr_mod(Nat.1, m)
        // Modulo m: the first term is a * (n v) ≡ a * 1 = a, the second
        // term is a multiple of m and hence ≡ 0.
        congr_mod_refl(a, m)
        a.congr_mod(a, m)
        congr_mod_mul(a, a, n * mod_inv(n, m), Nat.1, m)
        (a * (n * mod_inv(n, m))).congr_mod(a * Nat.1, m)
        mul_one_right(a)
        a * Nat.1 = a
        (a * (n * mod_inv(n, m))).congr_mod(a, m)
        mul_assoc(a, n, mod_inv(n, m))
        a * n * mod_inv(n, m) = a * (n * mod_inv(n, m))
        (a * n * mod_inv(n, m)).congr_mod(a, m)
        theorems1000_mul_congr_zero(b, m, mod_inv(m, n))
        (b * (m * mod_inv(m, n))).congr_mod(Nat.0, m)
        mul_assoc(b, m, mod_inv(m, n))
        b * m * mod_inv(m, n) = b * (m * mod_inv(m, n))
        (b * m * mod_inv(m, n)).congr_mod(Nat.0, m)
        congr_mod_add(a * n * mod_inv(n, m), b * m * mod_inv(m, n), a, Nat.0, m)
        (a * n * mod_inv(n, m) + b * m * mod_inv(m, n)).congr_mod(a + Nat.0, m)
        add_zero_right(a)
        a + Nat.0 = a
        (a * n * mod_inv(n, m) + b * m * mod_inv(m, n)).congr_mod(a, m)
        // Modulo n: the second term is b * (m u) ≡ b * 1 = b, the first
        // term is a multiple of n and hence ≡ 0.
        congr_mod_refl(b, n)
        b.congr_mod(b, n)
        congr_mod_mul(b, b, m * mod_inv(m, n), Nat.1, n)
        (b * (m * mod_inv(m, n))).congr_mod(b * Nat.1, n)
        mul_one_right(b)
        b * Nat.1 = b
        (b * (m * mod_inv(m, n))).congr_mod(b, n)
        mul_assoc(b, m, mod_inv(m, n))
        b * m * mod_inv(m, n) = b * (m * mod_inv(m, n))
        (b * m * mod_inv(m, n)).congr_mod(b, n)
        theorems1000_mul_congr_zero(a, n, mod_inv(n, m))
        (a * (n * mod_inv(n, m))).congr_mod(Nat.0, n)
        mul_assoc(a, n, mod_inv(n, m))
        a * n * mod_inv(n, m) = a * (n * mod_inv(n, m))
        (a * n * mod_inv(n, m)).congr_mod(Nat.0, n)
        congr_mod_add(b * m * mod_inv(m, n), a * n * mod_inv(n, m), b, Nat.0, n)
        (b * m * mod_inv(m, n) + a * n * mod_inv(n, m)).congr_mod(b + Nat.0, n)
        add_zero_right(b)
        b + Nat.0 = b
        (b * m * mod_inv(m, n) + a * n * mod_inv(n, m)).congr_mod(b, n)
        // The sum is commutative, so the addition-order swap is harmless.
        add_comm(a * n * mod_inv(n, m), b * m * mod_inv(m, n))
        a * n * mod_inv(n, m) + b * m * mod_inv(m, n) =
            b * m * mod_inv(m, n) + a * n * mod_inv(n, m)
        (a * n * mod_inv(n, m) + b * m * mod_inv(m, n)).congr_mod(b, n)
        // The witness c = a n v + b m u satisfies both congruences.
        exists(c: Nat) { c.congr_mod(a, m) and c.congr_mod(b, n) }
    }
}

/// The Chinese remainder theorem, uniqueness: two solutions of the same
/// pair of congruences are congruent modulo the product of the moduli.
theorem theorems1000_chinese_remainder_unique(
    m: Nat, n: Nat, c1: Nat, c2: Nat, a: Nat, b: Nat
) {
    m.coprime(n) and
    c1.congr_mod(a, m) and c2.congr_mod(a, m) and
    c1.congr_mod(b, n) and c2.congr_mod(b, n)
    implies c1.congr_mod(c2, m * n)
} by {
    if m.coprime(n) and
       c1.congr_mod(a, m) and c2.congr_mod(a, m) and
       c1.congr_mod(b, n) and c2.congr_mod(b, n) {
        // c1 ≡ c2 modulo m.
        congr_mod_symm(c2, a, m)
        a.congr_mod(c2, m)
        congr_mod_trans(c1, a, c2, m)
        c1.congr_mod(c2, m)
        // c1 ≡ c2 modulo n.
        congr_mod_symm(c2, b, n)
        b.congr_mod(c2, n)
        congr_mod_trans(c1, b, c2, n)
        c1.congr_mod(c2, n)
        // Combine the two congruences on the coprime moduli.
        nat_congr_combine_coprime(m, n, c1, c2)
        c1.congr_mod(c2, m * n)
    }
}
