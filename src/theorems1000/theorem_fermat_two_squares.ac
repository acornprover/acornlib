from nat import Nat, mul_assoc, mul_comm, mul_one_right, add_zero_right, add_cancels_left,
    divides_self, div_imp_mod, mod_of_zero, small_mod, mod_lt, mod_of_decomp, div_mod_decomp,
    div_of_decomp, pos_of_ne_zero, lt_suc_right, lt_suc, lt_trans, not_lt_zero, alt_suc_ne_zero
from number_theory.interface import congr_mod_refl, congr_mod_symm, congr_mod_trans,
    congr_mod_mul, congr_mod_add, mod_congr_mod_self

numerals Nat

// Fermat's theorem on sums of two squares (Pierre de Fermat, 1640; stated
// in a letter to Marin Mersenne; first published proof by Leonhard Euler,
// 1749, after seven years of work): an odd prime `p` can be written as a
// sum of two squares,
//     p = a² + b²   (a, b natural),
// exactly when `p ≡ 1 (mod 4)`.  The prime `2 = 1² + 1²` is the single
// even exception.  The theorem is the starting point of the theory of
// quadratic forms: an integer is a sum of two squares exactly when every
// prime congruent to 3 modulo 4 occurs in its factorization to an even
// exponent (the multiplicative half of this statement, the
// Brahmagupta–Fibonacci identity, is proved in
// theorem_two_square_closure.ac in this directory).
//
// The easy direction — a prime that is a sum of two squares and is not 2
// is congruent to 1 modulo 4 — is proved below.  It is a one-line
// observation about squares modulo 4: every square is 0 or 1 modulo 4, so
// a sum of two squares is 0, 1, or 2 modulo 4, and a prime (other than 2)
// is neither 0 nor 2 modulo 4.  The proof formalizes the three ingredients
// — the square lemma, the sum lemma, and the fact that a prime divisible
// by 2 is exactly 2 — with the congruence calculus of the public
// `number_theory` interface.
//
// The hard direction — every prime congruent to 1 modulo 4 is a sum of two
// squares — is recorded as a commented statement at the end.  Its classical
// proof runs: since `p ≡ 1 (mod 4)`, `-1` is a quadratic residue modulo
// `p` (Wilson's theorem, or Euler's criterion), say `x² ≡ -1 (mod p)`;
// Thue's lemma then produces `a, b` with `|a|, |b| < √p` and
// `a ≡ b x (mod p)`, so `a² + b²` is a positive multiple of `p` smaller
// than `2p`, hence exactly `p`.  Both directions are proved inside the
// library (`number_theory/sum_of_two_squares.ac`:
// `prime_sum_of_two_squares` and `prime_sum_two_squares_converse`), but
// that module is not part of the public `number_theory` interface, and the
// descent argument is not replayed here.

/// Four is greater than one.
theorem theorems1000_one_lt_four {
    Nat.1 < Nat.4
} by {
    lt_suc(Nat.1)
    Nat.1 < Nat.1.suc
    Nat.1.suc = Nat.2
    Nat.1 < Nat.2
    lt_suc(Nat.2)
    Nat.2 < Nat.2.suc
    Nat.2.suc = Nat.3
    Nat.2 < Nat.3
    lt_suc(Nat.3)
    Nat.3 < Nat.3.suc
    Nat.3.suc = Nat.4
    Nat.3 < Nat.4
    lt_trans(Nat.2, Nat.3, Nat.4)
    Nat.2 < Nat.4
    lt_trans(Nat.1, Nat.2, Nat.4)
    Nat.1 < Nat.4
}

/// Four is greater than two.
theorem theorems1000_two_lt_four {
    Nat.2 < Nat.4
} by {
    lt_suc(Nat.2)
    Nat.2 < Nat.2.suc
    Nat.2.suc = Nat.3
    Nat.2 < Nat.3
    lt_suc(Nat.3)
    Nat.3 < Nat.3.suc
    Nat.3.suc = Nat.4
    Nat.3 < Nat.4
    lt_trans(Nat.2, Nat.3, Nat.4)
    Nat.2 < Nat.4
}

/// Every square is congruent to zero or one modulo four.
theorem theorems1000_square_mod_four(a: Nat) {
    (a * a).mod(Nat.4) = Nat.0 or (a * a).mod(Nat.4) = Nat.1
} by {
    let r: Nat = a.mod(Nat.4)
    mod_congr_mod_self(a, Nat.4)
    a.mod(Nat.4).congr_mod(a, Nat.4)
    congr_mod_symm(a.mod(Nat.4), a, Nat.4)
    a.congr_mod(a.mod(Nat.4), Nat.4)
    a.congr_mod(r, Nat.4)
    congr_mod_mul(a, r, a, r, Nat.4)
    (a * a).congr_mod(r * r, Nat.4)
    (a * a).mod(Nat.4) = (r * r).mod(Nat.4)
    alt_suc_ne_zero(Nat.3)
    Nat.4 != Nat.0
    mod_lt(a, Nat.4)
    r < Nat.4
    lt_suc_right(r, Nat.3)
    if r != Nat.3 {
        r < Nat.3
        lt_suc_right(r, Nat.2)
        if r != Nat.2 {
            r < Nat.2
            lt_suc_right(r, Nat.1)
            if r != Nat.1 {
                r < Nat.1
                lt_suc_right(r, Nat.0)
                if r != Nat.0 {
                    r < Nat.0
                    not_lt_zero(r)
                    false
                }
                r = Nat.0
                r * r = Nat.0
                (a * a).mod(Nat.4) = Nat.0.mod(Nat.4)
                mod_of_zero(Nat.4)
                Nat.0.mod(Nat.4) = Nat.0
                (a * a).mod(Nat.4) = Nat.0
                (a * a).mod(Nat.4) = Nat.0 or (a * a).mod(Nat.4) = Nat.1
            } else {
                r = Nat.1
                r * r = Nat.1
                (a * a).mod(Nat.4) = Nat.1.mod(Nat.4)
                small_mod(Nat.1, Nat.4)
                Nat.1.mod(Nat.4) = Nat.1
                (a * a).mod(Nat.4) = Nat.1
                (a * a).mod(Nat.4) = Nat.0 or (a * a).mod(Nat.4) = Nat.1
            }
        } else {
            r = Nat.2
            r * r = Nat.4
            (a * a).mod(Nat.4) = Nat.4.mod(Nat.4)
            divides_self(Nat.4)
            Nat.4.divides(Nat.4)
            div_imp_mod(Nat.4, Nat.4)
            Nat.4.mod(Nat.4) = Nat.0
            (a * a).mod(Nat.4) = Nat.0
            (a * a).mod(Nat.4) = Nat.0 or (a * a).mod(Nat.4) = Nat.1
        }
    } else {
        r = Nat.3
        r * r = Nat.9
        theorems1000_one_lt_four
        Nat.1 < Nat.4
        div_of_decomp(Nat.2, Nat.1, Nat.4)
        (Nat.2 * Nat.4 + Nat.1).div(Nat.4) = Nat.2
        Nat.2 * Nat.4 + Nat.1 = Nat.9
        Nat.9.div(Nat.4) = Nat.2
        div_mod_decomp(Nat.9, Nat.4)
        Nat.9.div(Nat.4) * Nat.4 + Nat.9.mod(Nat.4) = Nat.9
        Nat.2 * Nat.4 + Nat.9.mod(Nat.4) = Nat.9
        Nat.2 * Nat.4 + Nat.1 = Nat.9
        add_cancels_left(Nat.2 * Nat.4, Nat.9.mod(Nat.4), Nat.1)
        Nat.9.mod(Nat.4) = Nat.1
        (a * a).mod(Nat.4) = Nat.9.mod(Nat.4)
        (a * a).mod(Nat.4) = Nat.1
        (a * a).mod(Nat.4) = Nat.0 or (a * a).mod(Nat.4) = Nat.1
    }
}

/// The sum of two squares is congruent to zero, one, or two modulo four.
theorem theorems1000_sum_two_squares_mod_four(a: Nat, b: Nat) {
    (a * a + b * b).mod(Nat.4) = Nat.0 or
        (a * a + b * b).mod(Nat.4) = Nat.1 or
        (a * a + b * b).mod(Nat.4) = Nat.2
} by {
    theorems1000_square_mod_four(a)
    (a * a).mod(Nat.4) = Nat.0 or (a * a).mod(Nat.4) = Nat.1
    theorems1000_square_mod_four(b)
    (b * b).mod(Nat.4) = Nat.0 or (b * b).mod(Nat.4) = Nat.1
    if (a * a).mod(Nat.4) != Nat.0 {
        (a * a).mod(Nat.4) = Nat.1
        (a * a).congr_mod(Nat.1, Nat.4)
        if (b * b).mod(Nat.4) != Nat.0 {
            (b * b).mod(Nat.4) = Nat.1
            (b * b).congr_mod(Nat.1, Nat.4)
            congr_mod_add(a * a, b * b, Nat.1, Nat.1, Nat.4)
            (a * a + b * b).congr_mod(Nat.2, Nat.4)
            (a * a + b * b).mod(Nat.4) = Nat.2.mod(Nat.4)
            theorems1000_two_lt_four
            Nat.2 < Nat.4
            small_mod(Nat.2, Nat.4)
            Nat.2.mod(Nat.4) = Nat.2
            (a * a + b * b).mod(Nat.4) = Nat.2
            (a * a + b * b).mod(Nat.4) = Nat.0 or
                (a * a + b * b).mod(Nat.4) = Nat.1 or
                (a * a + b * b).mod(Nat.4) = Nat.2
        } else {
            (b * b).mod(Nat.4) = Nat.0
            (b * b).congr_mod(Nat.0, Nat.4)
            congr_mod_add(a * a, b * b, Nat.1, Nat.0, Nat.4)
            (a * a + b * b).congr_mod(Nat.1, Nat.4)
            (a * a + b * b).mod(Nat.4) = Nat.1.mod(Nat.4)
            small_mod(Nat.1, Nat.4)
            Nat.1.mod(Nat.4) = Nat.1
            (a * a + b * b).mod(Nat.4) = Nat.1
            (a * a + b * b).mod(Nat.4) = Nat.0 or
                (a * a + b * b).mod(Nat.4) = Nat.1 or
                (a * a + b * b).mod(Nat.4) = Nat.2
        }
        (a * a + b * b).mod(Nat.4) = Nat.0 or
            (a * a + b * b).mod(Nat.4) = Nat.1 or
            (a * a + b * b).mod(Nat.4) = Nat.2
    } else {
        (a * a).mod(Nat.4) = Nat.0
        (a * a).congr_mod(Nat.0, Nat.4)
        if (b * b).mod(Nat.4) != Nat.0 {
            (b * b).mod(Nat.4) = Nat.1
            (b * b).congr_mod(Nat.1, Nat.4)
            congr_mod_add(a * a, b * b, Nat.0, Nat.1, Nat.4)
            (a * a + b * b).congr_mod(Nat.1, Nat.4)
            (a * a + b * b).mod(Nat.4) = Nat.1.mod(Nat.4)
            small_mod(Nat.1, Nat.4)
            Nat.1.mod(Nat.4) = Nat.1
            (a * a + b * b).mod(Nat.4) = Nat.1
            (a * a + b * b).mod(Nat.4) = Nat.0 or
                (a * a + b * b).mod(Nat.4) = Nat.1 or
                (a * a + b * b).mod(Nat.4) = Nat.2
        } else {
            (b * b).mod(Nat.4) = Nat.0
            (b * b).congr_mod(Nat.0, Nat.4)
            congr_mod_add(a * a, b * b, Nat.0, Nat.0, Nat.4)
            (a * a + b * b).congr_mod(Nat.0, Nat.4)
            (a * a + b * b).mod(Nat.4) = Nat.0.mod(Nat.4)
            mod_of_zero(Nat.4)
            Nat.0.mod(Nat.4) = Nat.0
            (a * a + b * b).mod(Nat.4) = Nat.0
            (a * a + b * b).mod(Nat.4) = Nat.0 or
                (a * a + b * b).mod(Nat.4) = Nat.1 or
                (a * a + b * b).mod(Nat.4) = Nat.2
        }
        (a * a + b * b).mod(Nat.4) = Nat.0 or
            (a * a + b * b).mod(Nat.4) = Nat.1 or
            (a * a + b * b).mod(Nat.4) = Nat.2
    }
}

/// A prime divisible by two is exactly two.
theorem theorems1000_prime_two_divides_imp_eq_two(p: Nat) {
    p.is_prime and Nat.2.divides(p) implies p = Nat.2
} by {
    if p.is_prime and Nat.2.divides(p) {
        p.is_prime = (Nat.1 < p and not p.is_composite)
        Nat.1 < p
        not p.is_composite
        p.is_composite = exists(b: Nat, c: Nat) { Nat.1 < b and Nat.1 < c and p = b * c }
        Nat.2.divides(p) = exists(c: Nat) { Nat.2 * c = p }
        exists(c: Nat) { Nat.2 * c = p }
        let c: Nat satisfy { Nat.2 * c = p }
        if c = Nat.1 {
            Nat.2 * c = p
            Nat.2 * Nat.1 = p
            Nat.2 * Nat.1 = Nat.2
            p = Nat.2
        }
        if not c = Nat.1 {
            if c = Nat.0 {
                Nat.2 * c = p
                Nat.2 * Nat.0 = p
                Nat.2 * Nat.0 = Nat.0
                p = Nat.0
                Nat.1 < p
                false
            }
            c != Nat.0
            pos_of_ne_zero(c)
            Nat.0 < c
            Nat.1 < c
            Nat.1 < Nat.2
            Nat.2 * c = p
            exists(b: Nat, c2: Nat) { Nat.1 < b and Nat.1 < c2 and p = b * c2 }
            p.is_composite
            false
        }
        p = Nat.2
    }
}

/// A number with remainder zero modulo four is divisible by two.
theorem theorems1000_mod_four_zero_imp_two_divides(p: Nat) {
    p.mod(Nat.4) = Nat.0 implies Nat.2.divides(p)
} by {
    if p.mod(Nat.4) = Nat.0 {
        div_mod_decomp(p, Nat.4)
        p.div(Nat.4) * Nat.4 + p.mod(Nat.4) = p
        p.div(Nat.4) * Nat.4 + Nat.0 = p
        p.div(Nat.4) * Nat.4 = p
        p = p.div(Nat.4) * Nat.4
        p = Nat.4 * p.div(Nat.4)
        Nat.4 = Nat.2 * Nat.2
        p = Nat.2 * Nat.2 * p.div(Nat.4)
        p = Nat.2 * (Nat.2 * p.div(Nat.4))
        exists(c: Nat) { Nat.2 * c = p }
        Nat.2.divides(p)
    }
}

/// `4 q + 2` factors as `2 (2 q + 1)`.
theorem theorems1000_mul_four_add_two(q: Nat) {
    q * Nat.4 + Nat.2 = Nat.2 * (Nat.2 * q + Nat.1)
} by {
    q * Nat.4 = Nat.4 * q
    Nat.4 = Nat.2 * Nat.2
    q * Nat.4 = Nat.2 * Nat.2 * q
    q * Nat.4 = Nat.2 * (Nat.2 * q)
    q * Nat.4 + Nat.2 = Nat.2 * (Nat.2 * q) + Nat.2
    Nat.2 * (Nat.2 * q) + Nat.2 = Nat.2 * (Nat.2 * q) + Nat.2 * Nat.1
    Nat.2 * (Nat.2 * q) + Nat.2 * Nat.1 = Nat.2 * (Nat.2 * q + Nat.1)
    q * Nat.4 + Nat.2 = Nat.2 * (Nat.2 * q + Nat.1)
}

/// A number with remainder two modulo four is divisible by two.
theorem theorems1000_mod_four_two_imp_two_divides(p: Nat) {
    p.mod(Nat.4) = Nat.2 implies Nat.2.divides(p)
} by {
    if p.mod(Nat.4) = Nat.2 {
        div_mod_decomp(p, Nat.4)
        p.div(Nat.4) * Nat.4 + p.mod(Nat.4) = p
        p.div(Nat.4) * Nat.4 + Nat.2 = p
        theorems1000_mul_four_add_two(p.div(Nat.4))
        p.div(Nat.4) * Nat.4 + Nat.2 = Nat.2 * (Nat.2 * p.div(Nat.4) + Nat.1)
        p = Nat.2 * (Nat.2 * p.div(Nat.4) + Nat.1)
        exists(c: Nat) { Nat.2 * c = p }
        Nat.2.divides(p)
    }
}

/// Fermat's theorem on sums of two squares, easy direction: an odd prime
/// that is a sum of two squares is congruent to one modulo four.
theorem theorems1000_fermat_two_squares_converse(p: Nat) {
    p.is_prime and p != Nat.2 and exists(a: Nat, b: Nat) { a * a + b * b = p }
        implies p.mod(Nat.4) = Nat.1
} by {
    if p.is_prime and p != Nat.2 and exists(a: Nat, b: Nat) { a * a + b * b = p } {
        let (a: Nat, b: Nat) satisfy { a * a + b * b = p }
        theorems1000_sum_two_squares_mod_four(a, b)
        (a * a + b * b).mod(Nat.4) = Nat.0 or
            (a * a + b * b).mod(Nat.4) = Nat.1 or
            (a * a + b * b).mod(Nat.4) = Nat.2
        a * a + b * b = p
        (a * a + b * b).mod(Nat.4) = p.mod(Nat.4)
        if (a * a + b * b).mod(Nat.4) != Nat.0 {
            if (a * a + b * b).mod(Nat.4) != Nat.2 {
                (a * a + b * b).mod(Nat.4) = Nat.1
                (a * a + b * b).mod(Nat.4) = p.mod(Nat.4)
                p.mod(Nat.4) = Nat.1
                p.mod(Nat.4) = Nat.1
            } else {
                (a * a + b * b).mod(Nat.4) = Nat.2
                p.mod(Nat.4) = Nat.2
                theorems1000_mod_four_two_imp_two_divides(p)
                Nat.2.divides(p)
                theorems1000_prime_two_divides_imp_eq_two(p)
                p = Nat.2
                p != Nat.2
                false
            }
            p.mod(Nat.4) = Nat.1
        } else {
            (a * a + b * b).mod(Nat.4) = Nat.0
            p.mod(Nat.4) = Nat.0
            theorems1000_mod_four_zero_imp_two_divides(p)
            Nat.2.divides(p)
            theorems1000_prime_two_divides_imp_eq_two(p)
            p = Nat.2
            p != Nat.2
            false
        }
        p.mod(Nat.4) = Nat.1
    }
}

// The hard direction of Fermat's theorem — every prime congruent to 1
// modulo 4 is a sum of two squares — is recorded below without proof; the
// descent argument (quadratic residue -1 modulo p, Thue's lemma, and the
// bound a² + b² < 2p) is sketched in the header comment of this file and
// is proved inside the library's `number_theory/sum_of_two_squares.ac`
// (`prime_sum_of_two_squares`), which is not part of the public interface.

// theorem theorems1000_fermat_two_squares_forward(p: Nat) {
//     p.is_prime and p.mod(Nat.4) = Nat.1 implies
//         exists(a: Nat, b: Nat) { a * a + b * b = p }
// }
