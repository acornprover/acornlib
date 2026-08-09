from pair import Pair
from real import Real
from top100 import r2_add, r2_neg, r2_sub, r2_dot, r2_norm_sq
from theorems1000.r2_helpers import r2_sub_through, r2_sub_reverse_neg, r2_norm_sq_neg,
    r2_norm_sq_add_expansion, r2_norm_sq_add_add_expand, r2_norm_sq_double,
    r2_double_sub_distrib, r2_sub_add_pair_distrib,
    real_add_six_pair_rearrange, real_add_pair_rearrange, real_add_four_last_next_to_first

// Euler's quadrilateral theorem: in any quadrilateral, the sum of the squares
// of the four sides equals the sum of the squares of the two diagonals plus
// four times the square of the segment joining the midpoints of the diagonals.
// With `m` the midpoint of the diagonal `ac` and `n` the midpoint of the
// diagonal `bd`, the statement is
//     |b-a|^2 + |c-b|^2 + |d-c|^2 + |a-d|^2
//         = |c-a|^2 + |d-b|^2 + 4 |n-m|^2,
// where `4 |n-m|^2` is written as `|(n-m)+(n-m)|^2`.
//
// The proof writes the side vector `a-d` as the negative of the sum of the
// other three side vectors, expands its squared norm with the three-term
// expansion, and expands the two diagonals and the doubled midpoint segment
// with the two-term expansion; all cross terms then cancel pairwise.

/// Rearrangement of the twelve terms of Euler's quadrilateral identity.
theorem real_euler_quad_rearrange(a: Real, b: Real, c: Real, x: Real, y: Real, z: Real) {
    a + b + c + (a + b + c + y + y + x + x + z + z) =
        (a + b + x + x) + (b + c + y + y) + (a + c + z + z)
} by {
    // --- flatten the left side ---
    a + b + c + (a + b + c + y + y + x + x + z + z) =
        a + b + c + (a + b + c + y + y + x + x + z) + z
    a + b + c + (a + b + c + y + y + x + x + z) + z =
        a + b + c + (a + b + c + y + y + x + x) + z + z
    a + b + c + (a + b + c + y + y + x + x) + z + z =
        a + b + c + (a + b + c + y + y + x) + x + z + z
    a + b + c + (a + b + c + y + y + x) + x + z + z =
        a + b + c + (a + b + c + y + y) + x + x + z + z
    a + b + c + (a + b + c + y + y) + x + x + z + z =
        a + b + c + (a + b + c + y) + y + x + x + z + z
    a + b + c + (a + b + c + y) + y + x + x + z + z =
        a + b + c + (a + b + c) + y + y + x + x + z + z
    a + b + c + (a + b + c) + y + y + x + x + z + z =
        a + b + c + (a + b) + c + y + y + x + x + z + z
    a + b + c + (a + b) + c + y + y + x + x + z + z =
        a + b + c + a + b + c + y + y + x + x + z + z
    // --- group the first six terms ---
    a + b + c + a + b + c + y + y + x + x + z + z =
        (a + b + c) + (a + b + c) + y + y + x + x + z + z
    real_add_six_pair_rearrange(a, b, c, a, b, c)
    (a + b + c) + (a + b + c) = (a + a) + (c + c) + (b + b)
    (a + b + c) + (a + b + c) + y + y + x + x + z + z =
        (a + a) + (c + c) + (b + b) + y + y + x + x + z + z
    // --- reorder the pairs of the left side ---
    (a + a) + (c + c) + (b + b) + y + y + x + x + z + z =
        (a + a) + ((c + c) + (b + b)) + y + y + x + x + z + z
    (a + a) + ((c + c) + (b + b)) + y + y + x + x + z + z =
        (a + a) + ((b + b) + (c + c)) + y + y + x + x + z + z
    (a + a) + ((b + b) + (c + c)) + y + y + x + x + z + z =
        (a + a) + (b + b) + (c + c) + y + y + x + x + z + z
    (a + a) + (b + b) + (c + c) + y + y + x + x + z + z =
        (a + a) + (b + b) + (c + c) + (y + y) + (x + x) + z + z
    (a + a) + (b + b) + (c + c) + (y + y) + (x + x) + z + z =
        (a + a) + (b + b) + (c + c) + ((y + y) + (x + x)) + z + z
    (a + a) + (b + b) + (c + c) + ((y + y) + (x + x)) + z + z =
        (a + a) + (b + b) + (c + c) + ((x + x) + (y + y)) + z + z
    (a + a) + (b + b) + (c + c) + ((x + x) + (y + y)) + z + z =
        (a + a) + (b + b) + (c + c) + (x + x) + (y + y) + z + z
    (a + a) + (b + b) + (c + c) + (x + x) + (y + y) + z + z =
        (a + a) + (b + b) + (c + c) + (x + x) + (y + y) + (z + z)
    // --- flatten the right side ---
    (a + b + x + x) + (b + c + y + y) + (a + c + z + z) =
        (a + b + x + x) + (b + c + y + y) + (a + c) + z + z
    (a + b + x + x) + (b + c + y + y) + (a + c) + z + z =
        (a + b + x + x) + (b + c + y + y) + a + c + z + z
    (a + b + x + x) + (b + c + y + y) + a + c + z + z =
        (a + b + x + x) + (b + c) + y + y + a + c + z + z
    (a + b + x + x) + (b + c) + y + y + a + c + z + z =
        (a + b + x + x) + b + c + y + y + a + c + z + z
    (a + b + x + x) + b + c + y + y + a + c + z + z =
        (a + b) + x + x + b + c + y + y + a + c + z + z
    (a + b) + x + x + b + c + y + y + a + c + z + z =
        a + b + x + x + b + c + y + y + a + c + z + z
    // --- regroup the right side into pairs ---
    a + b + x + x + b + c + y + y + a + c + z + z =
        (a + b) + (x + x) + (b + c) + (y + y) + (a + c) + (z + z)
    (a + b) + (x + x) + (b + c) + (y + y) + (a + c) + (z + z) =
        ((a + b) + (x + x)) + ((b + c) + (y + y)) + (a + c) + (z + z)
    real_add_pair_rearrange(a + b, x + x, b + c, y + y)
    ((a + b) + (x + x)) + ((b + c) + (y + y)) = ((a + b) + (b + c)) + ((x + x) + (y + y))
    ((a + b) + (x + x)) + ((b + c) + (y + y)) + (a + c) + (z + z) =
        ((a + b) + (b + c)) + ((x + x) + (y + y)) + (a + c) + (z + z)
    ((a + b) + (b + c)) + ((x + x) + (y + y)) + (a + c) + (z + z) =
        (a + b + b + c) + ((x + x) + (y + y)) + (a + c) + (z + z)
    real_add_four_last_next_to_first(a, b, b, c)
    a + b + b + c = (a + c) + (b + b)
    (a + b + b + c) + ((x + x) + (y + y)) + (a + c) + (z + z) =
        (a + c) + (b + b) + ((x + x) + (y + y)) + (a + c) + (z + z)
    // --- pair the two (a + c) blocks and finish ---
    (a + c) + (b + b) + ((x + x) + (y + y)) + (a + c) + (z + z) =
        (a + c) + (a + c) + (b + b) + ((x + x) + (y + y)) + (z + z)
    real_add_pair_rearrange(a, c, a, c)
    (a + c) + (a + c) = (a + a) + (c + c)
    (a + c) + (a + c) + (b + b) + ((x + x) + (y + y)) + (z + z) =
        (a + a) + (c + c) + (b + b) + ((x + x) + (y + y)) + (z + z)
    (a + a) + (c + c) + (b + b) + ((x + x) + (y + y)) + (z + z) =
        (a + a) + ((c + c) + (b + b)) + ((x + x) + (y + y)) + (z + z)
    (a + a) + ((c + c) + (b + b)) + ((x + x) + (y + y)) + (z + z) =
        (a + a) + ((b + b) + (c + c)) + ((x + x) + (y + y)) + (z + z)
    (a + a) + ((b + b) + (c + c)) + ((x + x) + (y + y)) + (z + z) =
        (a + a) + (b + b) + (c + c) + ((x + x) + (y + y)) + (z + z)
    (a + a) + (b + b) + (c + c) + ((x + x) + (y + y)) + (z + z) =
        (a + a) + (b + b) + (c + c) + (x + x) + (y + y) + (z + z)
}

/// Euler's quadrilateral theorem.
///
/// In any quadrilateral with vertices `a`, `b`, `c`, `d`, the sum of the
/// squared side lengths equals the sum of the squared diagonal lengths plus
/// four times the squared distance between the midpoints of the diagonals,
/// where `m` is the midpoint of `ac` and `n` is the midpoint of `bd`.
theorem theorems1000_euler_quadrilateral(
    a: Pair[Real, Real], b: Pair[Real, Real], c: Pair[Real, Real], d: Pair[Real, Real],
    m: Pair[Real, Real], n: Pair[Real, Real]
) {
    r2_add(m, m) = r2_add(a, c) and
    r2_add(n, n) = r2_add(b, d)
    implies
    r2_norm_sq(r2_sub(b, a)) + r2_norm_sq(r2_sub(c, b)) +
        r2_norm_sq(r2_sub(d, c)) + r2_norm_sq(r2_sub(a, d)) =
        r2_norm_sq(r2_sub(c, a)) + r2_norm_sq(r2_sub(d, b)) +
        r2_norm_sq(r2_add(r2_sub(n, m), r2_sub(n, m)))
} by {
    if r2_add(m, m) = r2_add(a, c) and r2_add(n, n) = r2_add(b, d) {
        r2_add(m, m) = r2_add(a, c)
        r2_add(n, n) = r2_add(b, d)
        // |a-d|^2 = |d-a|^2 and d-a = ((b-a)+(c-b))+(d-c)
        r2_sub_reverse_neg(a, d)
        r2_sub(a, d) = r2_neg(r2_sub(d, a))
        r2_norm_sq_neg(r2_sub(d, a))
        r2_norm_sq(r2_neg(r2_sub(d, a))) = r2_norm_sq(r2_sub(d, a))
        r2_norm_sq(r2_sub(a, d)) = r2_norm_sq(r2_sub(d, a))
        r2_sub_through(a, c, d)
        r2_sub(d, a) = r2_add(r2_sub(c, a), r2_sub(d, c))
        r2_sub_through(a, b, c)
        r2_sub(c, a) = r2_add(r2_sub(b, a), r2_sub(c, b))
        r2_sub(d, a) = r2_add(r2_add(r2_sub(b, a), r2_sub(c, b)), r2_sub(d, c))
        r2_norm_sq(r2_sub(a, d)) =
            r2_norm_sq(r2_add(r2_add(r2_sub(b, a), r2_sub(c, b)), r2_sub(d, c)))
        // the three-term expansion of |(b-a)+(c-b)+(d-c)|^2
        r2_norm_sq_add_add_expand(r2_sub(b, a), r2_sub(c, b), r2_sub(d, c))
        r2_norm_sq(r2_add(r2_add(r2_sub(b, a), r2_sub(c, b)), r2_sub(d, c))) =
            r2_norm_sq(r2_sub(b, a)) + r2_norm_sq(r2_sub(c, b)) + r2_norm_sq(r2_sub(d, c)) +
            r2_dot(r2_sub(c, b), r2_sub(d, c)) + r2_dot(r2_sub(c, b), r2_sub(d, c)) +
            r2_dot(r2_sub(b, a), r2_sub(c, b)) + r2_dot(r2_sub(b, a), r2_sub(d, c)) +
            r2_dot(r2_sub(b, a), r2_sub(c, b)) + r2_dot(r2_sub(b, a), r2_sub(d, c))
        r2_norm_sq(r2_sub(a, d)) =
            r2_norm_sq(r2_sub(b, a)) + r2_norm_sq(r2_sub(c, b)) + r2_norm_sq(r2_sub(d, c)) +
            r2_dot(r2_sub(c, b), r2_sub(d, c)) + r2_dot(r2_sub(c, b), r2_sub(d, c)) +
            r2_dot(r2_sub(b, a), r2_sub(c, b)) + r2_dot(r2_sub(b, a), r2_sub(d, c)) +
            r2_dot(r2_sub(b, a), r2_sub(c, b)) + r2_dot(r2_sub(b, a), r2_sub(d, c))
        // the left side
        r2_norm_sq(r2_sub(b, a)) + r2_norm_sq(r2_sub(c, b)) +
            r2_norm_sq(r2_sub(d, c)) + r2_norm_sq(r2_sub(a, d)) =
            r2_norm_sq(r2_sub(b, a)) + r2_norm_sq(r2_sub(c, b)) +
            r2_norm_sq(r2_sub(d, c)) +
            (r2_norm_sq(r2_sub(b, a)) + r2_norm_sq(r2_sub(c, b)) + r2_norm_sq(r2_sub(d, c)) +
            r2_dot(r2_sub(c, b), r2_sub(d, c)) + r2_dot(r2_sub(c, b), r2_sub(d, c)) +
            r2_dot(r2_sub(b, a), r2_sub(c, b)) + r2_dot(r2_sub(b, a), r2_sub(d, c)) +
            r2_dot(r2_sub(b, a), r2_sub(c, b)) + r2_dot(r2_sub(b, a), r2_sub(d, c)))
        // |c-a|^2 = |(b-a)+(c-b)|^2
        r2_sub_through(a, b, c)
        r2_sub(c, a) = r2_add(r2_sub(b, a), r2_sub(c, b))
        r2_norm_sq_add_expansion(r2_sub(b, a), r2_sub(c, b))
        r2_norm_sq(r2_add(r2_sub(b, a), r2_sub(c, b))) =
            r2_norm_sq(r2_sub(b, a)) + r2_norm_sq(r2_sub(c, b)) +
            r2_dot(r2_sub(b, a), r2_sub(c, b)) + r2_dot(r2_sub(b, a), r2_sub(c, b))
        r2_norm_sq(r2_sub(c, a)) =
            r2_norm_sq(r2_sub(b, a)) + r2_norm_sq(r2_sub(c, b)) +
            r2_dot(r2_sub(b, a), r2_sub(c, b)) + r2_dot(r2_sub(b, a), r2_sub(c, b))
        // |d-b|^2 = |(c-b)+(d-c)|^2
        r2_sub_through(b, c, d)
        r2_sub(d, b) = r2_add(r2_sub(c, b), r2_sub(d, c))
        r2_norm_sq_add_expansion(r2_sub(c, b), r2_sub(d, c))
        r2_norm_sq(r2_add(r2_sub(c, b), r2_sub(d, c))) =
            r2_norm_sq(r2_sub(c, b)) + r2_norm_sq(r2_sub(d, c)) +
            r2_dot(r2_sub(c, b), r2_sub(d, c)) + r2_dot(r2_sub(c, b), r2_sub(d, c))
        r2_norm_sq(r2_sub(d, b)) =
            r2_norm_sq(r2_sub(c, b)) + r2_norm_sq(r2_sub(d, c)) +
            r2_dot(r2_sub(c, b), r2_sub(d, c)) + r2_dot(r2_sub(c, b), r2_sub(d, c))
        // 2(n-m) = 2n - 2m = (b+d) - (a+c) = (b-a)+(d-c)
        r2_double_sub_distrib(n, m)
        r2_add(r2_sub(n, m), r2_sub(n, m)) = r2_sub(r2_add(n, n), r2_add(m, m))
        r2_sub(r2_add(n, n), r2_add(m, m)) = r2_sub(r2_add(b, d), r2_add(a, c))
        r2_sub_add_pair_distrib(b, d, a, c)
        r2_sub(r2_add(b, d), r2_add(a, c)) = r2_add(r2_sub(b, a), r2_sub(d, c))
        r2_add(r2_sub(n, m), r2_sub(n, m)) = r2_add(r2_sub(b, a), r2_sub(d, c))
        // 4|n-m|^2 = |2(n-m)|^2 = |(b-a)+(d-c)|^2
        r2_norm_sq_double(r2_sub(n, m))
        r2_norm_sq(r2_add(r2_sub(n, m), r2_sub(n, m))) =
            (Real.1 + Real.1) * ((Real.1 + Real.1) * r2_norm_sq(r2_sub(n, m)))
        r2_norm_sq(r2_add(r2_sub(n, m), r2_sub(n, m))) =
            r2_norm_sq(r2_add(r2_sub(b, a), r2_sub(d, c)))
        r2_norm_sq_add_expansion(r2_sub(b, a), r2_sub(d, c))
        r2_norm_sq(r2_add(r2_sub(b, a), r2_sub(d, c))) =
            r2_norm_sq(r2_sub(b, a)) + r2_norm_sq(r2_sub(d, c)) +
            r2_dot(r2_sub(b, a), r2_sub(d, c)) + r2_dot(r2_sub(b, a), r2_sub(d, c))
        r2_norm_sq(r2_add(r2_sub(n, m), r2_sub(n, m))) =
            r2_norm_sq(r2_sub(b, a)) + r2_norm_sq(r2_sub(d, c)) +
            r2_dot(r2_sub(b, a), r2_sub(d, c)) + r2_dot(r2_sub(b, a), r2_sub(d, c))
        // the right side
        r2_norm_sq(r2_sub(c, a)) + r2_norm_sq(r2_sub(d, b)) +
            r2_norm_sq(r2_add(r2_sub(n, m), r2_sub(n, m))) =
            (r2_norm_sq(r2_sub(b, a)) + r2_norm_sq(r2_sub(c, b)) +
                r2_dot(r2_sub(b, a), r2_sub(c, b)) + r2_dot(r2_sub(b, a), r2_sub(c, b))) +
            (r2_norm_sq(r2_sub(c, b)) + r2_norm_sq(r2_sub(d, c)) +
                r2_dot(r2_sub(c, b), r2_sub(d, c)) + r2_dot(r2_sub(c, b), r2_sub(d, c))) +
            (r2_norm_sq(r2_sub(b, a)) + r2_norm_sq(r2_sub(d, c)) +
                r2_dot(r2_sub(b, a), r2_sub(d, c)) + r2_dot(r2_sub(b, a), r2_sub(d, c)))
        // the twelve-term rearrangement
        real_euler_quad_rearrange(
            r2_norm_sq(r2_sub(b, a)),
            r2_norm_sq(r2_sub(c, b)),
            r2_norm_sq(r2_sub(d, c)),
            r2_dot(r2_sub(b, a), r2_sub(c, b)),
            r2_dot(r2_sub(c, b), r2_sub(d, c)),
            r2_dot(r2_sub(b, a), r2_sub(d, c)))
        r2_norm_sq(r2_sub(b, a)) + r2_norm_sq(r2_sub(c, b)) + r2_norm_sq(r2_sub(d, c)) +
            (r2_norm_sq(r2_sub(b, a)) + r2_norm_sq(r2_sub(c, b)) + r2_norm_sq(r2_sub(d, c)) +
            r2_dot(r2_sub(c, b), r2_sub(d, c)) + r2_dot(r2_sub(c, b), r2_sub(d, c)) +
            r2_dot(r2_sub(b, a), r2_sub(c, b)) + r2_dot(r2_sub(b, a), r2_sub(d, c)) +
            r2_dot(r2_sub(b, a), r2_sub(c, b)) + r2_dot(r2_sub(b, a), r2_sub(d, c))) =
            (r2_norm_sq(r2_sub(b, a)) + r2_norm_sq(r2_sub(c, b)) +
                r2_dot(r2_sub(b, a), r2_sub(c, b)) + r2_dot(r2_sub(b, a), r2_sub(c, b))) +
            (r2_norm_sq(r2_sub(c, b)) + r2_norm_sq(r2_sub(d, c)) +
                r2_dot(r2_sub(c, b), r2_sub(d, c)) + r2_dot(r2_sub(c, b), r2_sub(d, c))) +
            (r2_norm_sq(r2_sub(b, a)) + r2_norm_sq(r2_sub(d, c)) +
                r2_dot(r2_sub(b, a), r2_sub(d, c)) + r2_dot(r2_sub(b, a), r2_sub(d, c)))
        r2_norm_sq(r2_sub(b, a)) + r2_norm_sq(r2_sub(c, b)) +
            r2_norm_sq(r2_sub(d, c)) + r2_norm_sq(r2_sub(a, d)) =
            r2_norm_sq(r2_sub(c, a)) + r2_norm_sq(r2_sub(d, b)) +
            r2_norm_sq(r2_add(r2_sub(n, m), r2_sub(n, m)))
    }
}
