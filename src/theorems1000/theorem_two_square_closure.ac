from real import Real
from theorems1000.theorem_brahmagupta_fibonacci import theorems1000_brahmagupta_fibonacci

numerals Real

// The closure of the sums of two squares under multiplication: over the
// integers (and therefore over the reals), if `m = a^2 + b^2` and
// `n = c^2 + d^2` then the product `m n` is again a sum of two squares,
//     m n = (a c - b d)^2 + (a d + b c)^2.
// This is the multiplicative half of Fermat's theorem on sums of two
// squares (treated in `number_theory/sum_of_two_squares.ac`): combined
// with the fact that a prime congruent to 1 modulo 4 divides a sum of two
// squares, it shows that an integer is a sum of two squares exactly when
// every prime congruent to 3 modulo 4 occurs to an even exponent.  The
// underlying identity is the Brahmagupta-Fibonacci identity, proved in
// theorem_brahmagupta_fibonacci.ac in this directory; the theorems here
// package it in its number-theoretic, existential form, and iterate it to
// products of three factors.

/// The product of two sums of two squares is a sum of two squares.
theorem theorems1000_two_square_closure(m: Real, n: Real) {
    exists(a: Real, b: Real) { m = a * a + b * b } and
    exists(c: Real, d: Real) { n = c * c + d * d }
        implies exists(x: Real, y: Real) { m * n = x * x + y * y }
} by {
    if exists(a: Real, b: Real) { m = a * a + b * b } and
        exists(c: Real, d: Real) { n = c * c + d * d } {
        let (a: Real, b: Real) satisfy { m = a * a + b * b }
        let (c: Real, d: Real) satisfy { n = c * c + d * d }
        theorems1000_brahmagupta_fibonacci(a, b, c, d)
        (a * a + b * b) * (c * c + d * d) =
            (a * c - b * d) * (a * c - b * d) + (a * d + b * c) * (a * d + b * c)
        m * n = (a * c - b * d) * (a * c - b * d) + (a * d + b * c) * (a * d + b * c)
        exists(x: Real, y: Real) {
            x = a * c - b * d and y = a * d + b * c and m * n = x * x + y * y
        }
    }
}

/// The product of three sums of two squares is a sum of two squares, with
/// explicit witnesses: applying the two-square identity twice gives
/// `(a c - b d, a d + b c)` and then the composite pair.
theorem theorems1000_three_square_product(
    a: Real, b: Real, c: Real, d: Real, e: Real, f: Real
) {
    (a * a + b * b) * (c * c + d * d) * (e * e + f * f) =
        ((a * c - b * d) * e - (a * d + b * c) * f) *
            ((a * c - b * d) * e - (a * d + b * c) * f) +
        ((a * c - b * d) * f + (a * d + b * c) * e) *
            ((a * c - b * d) * f + (a * d + b * c) * e)
} by {
    theorems1000_brahmagupta_fibonacci(a, b, c, d)
    (a * a + b * b) * (c * c + d * d) =
        (a * c - b * d) * (a * c - b * d) + (a * d + b * c) * (a * d + b * c)
    (a * a + b * b) * (c * c + d * d) * (e * e + f * f) =
        ((a * c - b * d) * (a * c - b * d) + (a * d + b * c) * (a * d + b * c)) * (e * e + f * f)
    theorems1000_brahmagupta_fibonacci(a * c - b * d, a * d + b * c, e, f)
    ((a * c - b * d) * (a * c - b * d) + (a * d + b * c) * (a * d + b * c)) * (e * e + f * f) =
        ((a * c - b * d) * e - (a * d + b * c) * f) *
            ((a * c - b * d) * e - (a * d + b * c) * f) +
        ((a * c - b * d) * f + (a * d + b * c) * e) *
            ((a * c - b * d) * f + (a * d + b * c) * e)
}
