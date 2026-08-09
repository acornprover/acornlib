from pair import Pair, pair_ext
from real import Real
from top100 import r2_add, r2_neg, r2_sub, r2_dot, r2_norm_sq
from theorems1000.r2_helpers import r2_smul, r2_sub_add_pair_distrib, r2_smul_add_distrib,
    r2_smul_sub_distrib, r2_dot_add_right, r2_dot_smul_right, r2_sub_sub_same_base,
    r2_norm_sq_sub_expansion, r2_sub_eq_add_neg, r2_add_neg_right, r2_sub_reverse_neg,
    r2_norm_sq_neg, real_add_pair_rearrange

// Stewart's theorem: in a triangle with side `bc` divided at `d` into the
// parts `BD = m` and `DC = n`, the cevian `AD = w` satisfies
//     AC^2 * m + AB^2 * n = BC * (w^2 + m * n),
// where `BC = m + n`.  This is the classical identity
// `b^2 m + c^2 n = a (d^2 + m n)`.
//
// As in the library's Ptolemy and Heron statements, the lengths are given
// as witnesses: `m^2 = |b-d|^2`, `n^2 = |c-d|^2`, `w^2 = |a-d|^2`,
// `c_len^2 = |b-a|^2` (the side `AB = c`) and `b_len^2 = |c-a|^2` (the side
// `AC = b`).  The division point hypothesis is `n (b-d) + m (c-d) = 0`,
// written with scalar multiplication as
//     n b + m c = (m + n) d.
//
// Proof sketch: with `u = b - d`, `v = c - d` and `t = a - d`, the division
// hypothesis reads `n u + m v = 0`, so dotting with `t` gives
// `m (t . v) + n (t . u) = 0`.  The squared-difference identity expands
//     |c - a|^2 = |t - v|^2 = w^2 + n^2 - 2 (t . v),
//     |b - a|^2 = |t - u|^2 = w^2 + m^2 - 2 (t . u),
// and substituting these into `|c-a|^2 m + |b-a|^2 n` and cancelling
// `m (t . v) + n (t . u) = 0` leaves `(m + n) w^2 + m n (m + n)`, which is
// `(m + n) (w^2 + m n)`.

/// A scalar may be distributed over a coordinate pair in the scalar slot:
/// `k1 a + k2 a = (k1 + k2) a`.
theorem r2_smul_add_scalar(k1: Real, k2: Real, a: Pair[Real, Real]) {
    r2_add(r2_smul(k1, a), r2_smul(k2, a)) = r2_smul(k1 + k2, a)
} by {
    let sa1 = r2_smul(k1, a)
    let sa2 = r2_smul(k2, a)
    let lhs = r2_add(sa1, sa2)
    let rhs = r2_smul(k1 + k2, a)
    sa1.first = k1 * a.first
    sa1.second = k1 * a.second
    sa2.first = k2 * a.first
    sa2.second = k2 * a.second
    lhs.first = sa1.first + sa2.first
    lhs.first = k1 * a.first + k2 * a.first
    k1 * a.first + k2 * a.first = (k1 + k2) * a.first
    rhs.first = (k1 + k2) * a.first
    lhs.first = rhs.first
    lhs.second = sa1.second + sa2.second
    lhs.second = k1 * a.second + k2 * a.second
    k1 * a.second + k2 * a.second = (k1 + k2) * a.second
    rhs.second = (k1 + k2) * a.second
    lhs.second = rhs.second
    pair_ext(lhs, rhs)
}

/// Subtracting a point from itself gives the origin.
theorem r2_sub_self_zero(a: Pair[Real, Real]) {
    r2_sub(a, a) = Pair.new(Real.0, Real.0)
} by {
    r2_sub_eq_add_neg(a, a)
    r2_sub(a, a) = r2_add(a, r2_neg(a))
    r2_add_neg_right(a)
    r2_add(a, r2_neg(a)) = Pair.new(Real.0, Real.0)
    r2_sub(a, a) = Pair.new(Real.0, Real.0)
}

/// From the division hypothesis `n b + m c = (m + n) d`, the vector
/// combination `n (b - d) + m (c - d)` vanishes.
theorem stewart_vector_identity(
    b: Pair[Real, Real], c: Pair[Real, Real], d: Pair[Real, Real],
    m: Real, n: Real
) {
    r2_add(r2_smul(n, b), r2_smul(m, c)) = r2_smul(m + n, d) implies
    r2_add(r2_smul(n, r2_sub(b, d)), r2_smul(m, r2_sub(c, d))) = Pair.new(Real.0, Real.0)
} by {
    if r2_add(r2_smul(n, b), r2_smul(m, c)) = r2_smul(m + n, d) {
        r2_smul_sub_distrib(n, b, d)
        r2_smul(n, r2_sub(b, d)) = r2_sub(r2_smul(n, b), r2_smul(n, d))
        r2_smul_sub_distrib(m, c, d)
        r2_smul(m, r2_sub(c, d)) = r2_sub(r2_smul(m, c), r2_smul(m, d))
        r2_sub_add_pair_distrib(r2_smul(n, b), r2_smul(n, d), r2_smul(m, c), r2_smul(m, d))
        r2_sub(r2_add(r2_smul(n, b), r2_smul(m, c)), r2_add(r2_smul(n, d), r2_smul(m, d))) =
            r2_add(r2_sub(r2_smul(n, b), r2_smul(n, d)), r2_sub(r2_smul(m, c), r2_smul(m, d)))
        r2_add(r2_smul(n, r2_sub(b, d)), r2_smul(m, r2_sub(c, d))) =
            r2_sub(r2_add(r2_smul(n, b), r2_smul(m, c)), r2_add(r2_smul(n, d), r2_smul(m, d)))
        r2_add(r2_smul(n, r2_sub(b, d)), r2_smul(m, r2_sub(c, d))) =
            r2_sub(r2_smul(m + n, d), r2_add(r2_smul(n, d), r2_smul(m, d)))
        r2_smul_add_scalar(n, m, d)
        r2_add(r2_smul(n, d), r2_smul(m, d)) = r2_smul(n + m, d)
        r2_add(r2_smul(n, r2_sub(b, d)), r2_smul(m, r2_sub(c, d))) =
            r2_sub(r2_smul(m + n, d), r2_smul(n + m, d))
        r2_sub_self_zero(r2_smul(m + n, d))
        r2_sub(r2_smul(m + n, d), r2_smul(m + n, d)) = Pair.new(Real.0, Real.0)
        r2_add(r2_smul(n, r2_sub(b, d)), r2_smul(m, r2_sub(c, d))) = Pair.new(Real.0, Real.0)
    }
}

/// Dotting the vanishing combination with `t = a - d`:
/// `m (t . v) + n (t . u) = 0`.
theorem stewart_dot_zero(
    a: Pair[Real, Real], b: Pair[Real, Real], c: Pair[Real, Real], d: Pair[Real, Real],
    m: Real, n: Real
) {
    r2_add(r2_smul(n, r2_sub(b, d)), r2_smul(m, r2_sub(c, d))) = Pair.new(Real.0, Real.0) implies
    m * r2_dot(r2_sub(a, d), r2_sub(c, d)) + n * r2_dot(r2_sub(a, d), r2_sub(b, d)) = Real.0
} by {
    if r2_add(r2_smul(n, r2_sub(b, d)), r2_smul(m, r2_sub(c, d))) = Pair.new(Real.0, Real.0) {
        // the dot product with the origin vanishes
        r2_dot(r2_sub(a, d), Pair.new(Real.0, Real.0)) =
            r2_sub(a, d).first * Real.0 + r2_sub(a, d).second * Real.0
        r2_sub(a, d).first * Real.0 = Real.0
        r2_sub(a, d).second * Real.0 = Real.0
        r2_dot(r2_sub(a, d), Pair.new(Real.0, Real.0)) = Real.0 + Real.0
        Real.0 + Real.0 = Real.0
        r2_dot(r2_sub(a, d), Pair.new(Real.0, Real.0)) = Real.0
        r2_dot(r2_sub(a, d), r2_add(r2_smul(n, r2_sub(b, d)), r2_smul(m, r2_sub(c, d)))) = Real.0
        r2_dot_add_right(r2_sub(a, d), r2_smul(n, r2_sub(b, d)), r2_smul(m, r2_sub(c, d)))
        r2_dot(r2_sub(a, d), r2_add(r2_smul(n, r2_sub(b, d)), r2_smul(m, r2_sub(c, d)))) =
            r2_dot(r2_sub(a, d), r2_smul(n, r2_sub(b, d))) + r2_dot(r2_sub(a, d), r2_smul(m, r2_sub(c, d)))
        r2_dot_smul_right(n, r2_sub(a, d), r2_sub(b, d))
        r2_dot(r2_sub(a, d), r2_smul(n, r2_sub(b, d))) = n * r2_dot(r2_sub(a, d), r2_sub(b, d))
        r2_dot_smul_right(m, r2_sub(a, d), r2_sub(c, d))
        r2_dot(r2_sub(a, d), r2_smul(m, r2_sub(c, d))) = m * r2_dot(r2_sub(a, d), r2_sub(c, d))
        n * r2_dot(r2_sub(a, d), r2_sub(b, d)) + m * r2_dot(r2_sub(a, d), r2_sub(c, d)) = Real.0
        m * r2_dot(r2_sub(a, d), r2_sub(c, d)) + n * r2_dot(r2_sub(a, d), r2_sub(b, d)) = Real.0
    }
}

/// The squared side `AB` expands around the cevian foot:
/// `c_len^2 = w^2 + m^2 - 2 (t . u)` with `t = a - d`, `u = b - d`.
theorem stewart_expand_ab(
    a: Pair[Real, Real], b: Pair[Real, Real], d: Pair[Real, Real],
    w: Real, m: Real, c_len: Real
) {
    w * w = r2_norm_sq(r2_sub(a, d)) and
    m * m = r2_norm_sq(r2_sub(b, d)) and
    c_len * c_len = r2_norm_sq(r2_sub(b, a))
    implies
    c_len * c_len = w * w + m * m - r2_dot(r2_sub(a, d), r2_sub(b, d)) - r2_dot(r2_sub(a, d), r2_sub(b, d))
} by {
    if w * w = r2_norm_sq(r2_sub(a, d)) and
        m * m = r2_norm_sq(r2_sub(b, d)) and
        c_len * c_len = r2_norm_sq(r2_sub(b, a)) {
        w * w = r2_norm_sq(r2_sub(a, d))
        m * m = r2_norm_sq(r2_sub(b, d))
        c_len * c_len = r2_norm_sq(r2_sub(b, a))
        r2_sub_sub_same_base(d, b, a)
        r2_sub(r2_sub(a, d), r2_sub(b, d)) = r2_sub(a, b)
        r2_norm_sq_sub_expansion(r2_sub(a, d), r2_sub(b, d))
        r2_norm_sq(r2_sub(r2_sub(a, d), r2_sub(b, d))) =
            r2_norm_sq(r2_sub(a, d)) + r2_norm_sq(r2_sub(b, d)) -
            r2_dot(r2_sub(a, d), r2_sub(b, d)) - r2_dot(r2_sub(a, d), r2_sub(b, d))
        r2_norm_sq(r2_sub(a, b)) =
            r2_norm_sq(r2_sub(a, d)) + r2_norm_sq(r2_sub(b, d)) -
            r2_dot(r2_sub(a, d), r2_sub(b, d)) - r2_dot(r2_sub(a, d), r2_sub(b, d))
        r2_sub_reverse_neg(a, b)
        r2_sub(a, b) = r2_neg(r2_sub(b, a))
        r2_norm_sq_neg(r2_sub(b, a))
        r2_norm_sq(r2_neg(r2_sub(b, a))) = r2_norm_sq(r2_sub(b, a))
        r2_norm_sq(r2_sub(a, b)) = r2_norm_sq(r2_sub(b, a))
        r2_norm_sq(r2_sub(b, a)) =
            r2_norm_sq(r2_sub(a, d)) + r2_norm_sq(r2_sub(b, d)) -
            r2_dot(r2_sub(a, d), r2_sub(b, d)) - r2_dot(r2_sub(a, d), r2_sub(b, d))
        c_len * c_len = w * w + m * m - r2_dot(r2_sub(a, d), r2_sub(b, d)) - r2_dot(r2_sub(a, d), r2_sub(b, d))
    }
}

/// The squared side `AC` expands around the cevian foot:
/// `b_len^2 = w^2 + n^2 - 2 (t . v)` with `t = a - d`, `v = c - d`.
theorem stewart_expand_ac(
    a: Pair[Real, Real], c: Pair[Real, Real], d: Pair[Real, Real],
    w: Real, n: Real, b_len: Real
) {
    w * w = r2_norm_sq(r2_sub(a, d)) and
    n * n = r2_norm_sq(r2_sub(c, d)) and
    b_len * b_len = r2_norm_sq(r2_sub(c, a))
    implies
    b_len * b_len = w * w + n * n - r2_dot(r2_sub(a, d), r2_sub(c, d)) - r2_dot(r2_sub(a, d), r2_sub(c, d))
} by {
    if w * w = r2_norm_sq(r2_sub(a, d)) and
        n * n = r2_norm_sq(r2_sub(c, d)) and
        b_len * b_len = r2_norm_sq(r2_sub(c, a)) {
        w * w = r2_norm_sq(r2_sub(a, d))
        n * n = r2_norm_sq(r2_sub(c, d))
        b_len * b_len = r2_norm_sq(r2_sub(c, a))
        r2_sub_sub_same_base(d, c, a)
        r2_sub(r2_sub(a, d), r2_sub(c, d)) = r2_sub(a, c)
        r2_norm_sq_sub_expansion(r2_sub(a, d), r2_sub(c, d))
        r2_norm_sq(r2_sub(r2_sub(a, d), r2_sub(c, d))) =
            r2_norm_sq(r2_sub(a, d)) + r2_norm_sq(r2_sub(c, d)) -
            r2_dot(r2_sub(a, d), r2_sub(c, d)) - r2_dot(r2_sub(a, d), r2_sub(c, d))
        r2_norm_sq(r2_sub(a, c)) =
            r2_norm_sq(r2_sub(a, d)) + r2_norm_sq(r2_sub(c, d)) -
            r2_dot(r2_sub(a, d), r2_sub(c, d)) - r2_dot(r2_sub(a, d), r2_sub(c, d))
        r2_sub_reverse_neg(a, c)
        r2_sub(a, c) = r2_neg(r2_sub(c, a))
        r2_norm_sq_neg(r2_sub(c, a))
        r2_norm_sq(r2_neg(r2_sub(c, a))) = r2_norm_sq(r2_sub(c, a))
        r2_norm_sq(r2_sub(a, c)) = r2_norm_sq(r2_sub(c, a))
        r2_norm_sq(r2_sub(c, a)) =
            r2_norm_sq(r2_sub(a, d)) + r2_norm_sq(r2_sub(c, d)) -
            r2_dot(r2_sub(a, d), r2_sub(c, d)) - r2_dot(r2_sub(a, d), r2_sub(c, d))
        b_len * b_len = w * w + n * n - r2_dot(r2_sub(a, d), r2_sub(c, d)) - r2_dot(r2_sub(a, d), r2_sub(c, d))
    }
}

/// The real-algebra core of Stewart's theorem: given
/// `m (t . v) + n (t . u) = 0`, the identity
/// `(w^2 + n^2 - 2 (t . v)) m + (w^2 + m^2 - 2 (t . u)) n
///     = (m + n) (w^2 + m n)` holds.
theorem real_stewart_core(w2: Real, m: Real, n: Real, tu: Real, tv: Real) {
    m * tv + n * tu = Real.0 implies
    (w2 + n * n - tv - tv) * m + (w2 + m * m - tu - tu) * n = (m + n) * (w2 + m * n)
} by {
    if m * tv + n * tu = Real.0 {
        // ---- expand the first product ((w2 + n^2) - tv - tv) * m ----
        (w2 + n * n - tv - tv) * m = ((w2 + n * n) - tv) * m - tv * m
        ((w2 + n * n) - tv) * m = (w2 + n * n) * m - tv * m
        (w2 + n * n) * m = w2 * m + n * n * m
        ((w2 + n * n) - tv) * m = w2 * m + n * n * m - tv * m
        (w2 + n * n - tv - tv) * m = (w2 * m + n * n * m - tv * m) - tv * m
        (w2 + n * n - tv - tv) * m = w2 * m + n * n * m - tv * m - tv * m
        // ---- expand the second product ((w2 + m^2) - tu - tu) * n ----
        (w2 + m * m - tu - tu) * n = ((w2 + m * m) - tu) * n - tu * n
        ((w2 + m * m) - tu) * n = (w2 + m * m) * n - tu * n
        (w2 + m * m) * n = w2 * n + m * m * n
        ((w2 + m * m) - tu) * n = w2 * n + m * m * n - tu * n
        (w2 + m * m - tu - tu) * n = (w2 * n + m * m * n - tu * n) - tu * n
        (w2 + m * m - tu - tu) * n = w2 * n + m * m * n - tu * n - tu * n
        // ---- the sum of the two expansions ----
        (w2 + n * n - tv - tv) * m + (w2 + m * m - tu - tu) * n =
            (w2 * m + n * n * m - tv * m - tv * m) + (w2 + m * m - tu - tu) * n
        (w2 * m + n * n * m - tv * m - tv * m) + (w2 + m * m - tu - tu) * n =
            (w2 * m + n * n * m - tv * m - tv * m) + (w2 * n + m * m * n - tu * n - tu * n)
        (w2 + n * n - tv - tv) * m + (w2 + m * m - tu - tu) * n =
            (w2 * m + n * n * m - tv * m - tv * m) + (w2 * n + m * m * n - tu * n - tu * n)
        // ---- regroup the eight terms into three groups ----
        // write each four-term block as (first two) + (last two)
        w2 * m + n * n * m - tv * m - tv * m = (w2 * m + n * n * m - tv * m) + -(tv * m)
        (w2 * m + n * n * m - tv * m) + -(tv * m) = (w2 * m + n * n * m + -(tv * m)) + -(tv * m)
        (w2 * m + n * n * m + -(tv * m)) + -(tv * m) = (w2 * m + n * n * m) + (-(tv * m) + -(tv * m))
        w2 * n + m * m * n - tu * n - tu * n = (w2 * n + m * m * n - tu * n) + -(tu * n)
        (w2 * n + m * m * n - tu * n) + -(tu * n) = (w2 * n + m * m * n + -(tu * n)) + -(tu * n)
        (w2 * n + m * m * n + -(tu * n)) + -(tu * n) = (w2 * n + m * m * n) + (-(tu * n) + -(tu * n))
        // substitute the two block rewrites into the sum of the blocks
        (w2 * m + n * n * m - tv * m - tv * m) + (w2 * n + m * m * n - tu * n - tu * n) =
            ((w2 * m + n * n * m) + (-(tv * m) + -(tv * m))) + (w2 * n + m * m * n - tu * n - tu * n)
        ((w2 * m + n * n * m) + (-(tv * m) + -(tv * m))) + (w2 * n + m * m * n - tu * n - tu * n) =
            ((w2 * m + n * n * m) + (-(tv * m) + -(tv * m))) + ((w2 * n + m * m * n) + (-(tu * n) + -(tu * n)))
        // pair the blocks
        real_add_pair_rearrange(w2 * m + n * n * m, -(tv * m) + -(tv * m), w2 * n + m * m * n, -(tu * n) + -(tu * n))
        ((w2 * m + n * n * m) + (-(tv * m) + -(tv * m))) + ((w2 * n + m * m * n) + (-(tu * n) + -(tu * n))) =
            ((w2 * m + n * n * m) + (w2 * n + m * m * n)) + ((-(tv * m) + -(tv * m)) + (-(tu * n) + -(tu * n)))
        real_add_pair_rearrange(w2 * m, n * n * m, w2 * n, m * m * n)
        (w2 * m + n * n * m) + (w2 * n + m * m * n) = (w2 * m + w2 * n) + (n * n * m + m * m * n)
        (-(tv * m) + -(tv * m)) + (-(tu * n) + -(tu * n)) = -(tv * m) + (-(tv * m) + (-(tu * n) + -(tu * n)))
        ((w2 * m + n * n * m) + (w2 * n + m * m * n)) + ((-(tv * m) + -(tv * m)) + (-(tu * n) + -(tu * n))) =
            ((w2 * m + w2 * n) + (n * n * m + m * m * n)) + (-(tv * m) + (-(tv * m) + (-(tu * n) + -(tu * n))))
        (w2 * m + n * n * m - tv * m - tv * m) + (w2 * n + m * m * n - tu * n - tu * n) =
            (w2 * m + w2 * n) + (n * n * m + m * m * n) + (-(tv * m) + (-(tv * m) + (-(tu * n) + -(tu * n))))
        // ---- the cross terms: tv m + tu n = m tv + n tu = 0 ----
        tv * m = m * tv
        tu * n = n * tu
        tv * m + tu * n = m * tv + n * tu
        tv * m + tu * n = Real.0
        (tv * m + tu * n) + (tv * m + tu * n) = Real.0
        (tv * m + tu * n) + (tv * m + tu * n) = tv * m + tv * m + tu * n + tu * n
        tv * m + tv * m + tu * n + tu * n = Real.0
        // the negated cross terms vanish
        -(tv * m + tu * n) = Real.0
        -(tv * m + tu * n) = -(tv * m) + -(tu * n)
        -(tv * m) + -(tu * n) = Real.0
        (-(tv * m) + -(tu * n)) + (-(tv * m) + -(tu * n)) = Real.0
        (-(tv * m) + -(tu * n)) + (-(tv * m) + -(tu * n)) =
            -(tv * m) + (-(tv * m) + (-(tu * n) + -(tu * n)))
        -(tv * m) + (-(tv * m) + (-(tu * n) + -(tu * n))) = Real.0
        // ---- the square terms: n^2 m + m^2 n = m n (m + n) ----
        n * n * m = m * n * n
        m * m * n = m * n * m
        m * n * n + m * n * m = m * n * (n + m)
        n * n * m + m * m * n = m * n * (n + m)
        m * n * (n + m) = m * n * (m + n)
        n * n * m + m * m * n = m * n * (m + n)
        // ---- assemble the three groups ----
        (w2 * m + w2 * n) + (n * n * m + m * m * n) + (-(tv * m) + (-(tv * m) + (-(tu * n) + -(tu * n)))) =
            (w2 * m + w2 * n) + m * n * (m + n) + (-(tv * m) + (-(tv * m) + (-(tu * n) + -(tu * n))))
        (w2 * m + w2 * n) + m * n * (m + n) + (-(tv * m) + (-(tv * m) + (-(tu * n) + -(tu * n)))) =
            (w2 * m + w2 * n) + m * n * (m + n)
        w2 * m + w2 * n = (m + n) * w2
        (w2 * m + w2 * n) + m * n * (m + n) = (m + n) * w2 + m * n * (m + n)
        (m + n) * w2 + m * n * (m + n) = (m + n) * (w2 + m * n)
        (w2 + n * n - tv - tv) * m + (w2 + m * m - tu - tu) * n = (m + n) * (w2 + m * n)
    }
}

/// Stewart's theorem: the squared lengths of the two sides through `a`
/// combine with the parts `m`, `n` of the divided side as
/// `AC^2 * m + AB^2 * n = BC * (AD^2 + m * n)`.
///
/// With `d` on `bc` dividing it as `BD = m` and `DC = n` (the hypothesis
/// `n (b - d) + m (c - d) = 0`, i.e. `n b + m c = (m + n) d`), the cevian
/// `AD = w`, and the side witnesses `c_len^2 = |b-a|^2` (`AB = c`) and
/// `b_len^2 = |c-a|^2` (`AC = b`), the theorem reads
/// `b_len^2 * m + c_len^2 * n = (m + n) * (w^2 + m * n)`.
theorem theorems1000_stewart(
    a: Pair[Real, Real], b: Pair[Real, Real], c: Pair[Real, Real], d: Pair[Real, Real],
    b_len: Real, c_len: Real, m: Real, n: Real, w: Real
) {
    r2_add(r2_smul(n, b), r2_smul(m, c)) = r2_smul(m + n, d) and
    m * m = r2_norm_sq(r2_sub(b, d)) and
    n * n = r2_norm_sq(r2_sub(c, d)) and
    w * w = r2_norm_sq(r2_sub(a, d)) and
    c_len * c_len = r2_norm_sq(r2_sub(b, a)) and
    b_len * b_len = r2_norm_sq(r2_sub(c, a))
    implies
    b_len * b_len * m + c_len * c_len * n = (m + n) * (w * w + m * n)
} by {
    if r2_add(r2_smul(n, b), r2_smul(m, c)) = r2_smul(m + n, d) and
        m * m = r2_norm_sq(r2_sub(b, d)) and
        n * n = r2_norm_sq(r2_sub(c, d)) and
        w * w = r2_norm_sq(r2_sub(a, d)) and
        c_len * c_len = r2_norm_sq(r2_sub(b, a)) and
        b_len * b_len = r2_norm_sq(r2_sub(c, a)) {
        r2_add(r2_smul(n, b), r2_smul(m, c)) = r2_smul(m + n, d)
        m * m = r2_norm_sq(r2_sub(b, d))
        n * n = r2_norm_sq(r2_sub(c, d))
        w * w = r2_norm_sq(r2_sub(a, d))
        c_len * c_len = r2_norm_sq(r2_sub(b, a))
        b_len * b_len = r2_norm_sq(r2_sub(c, a))
        // the vector identity and its dot-product consequence
        stewart_vector_identity(b, c, d, m, n)
        r2_add(r2_smul(n, r2_sub(b, d)), r2_smul(m, r2_sub(c, d))) = Pair.new(Real.0, Real.0)
        stewart_dot_zero(a, b, c, d, m, n)
        m * r2_dot(r2_sub(a, d), r2_sub(c, d)) + n * r2_dot(r2_sub(a, d), r2_sub(b, d)) = Real.0
        // the two side expansions
        stewart_expand_ab(a, b, d, w, m, c_len)
        c_len * c_len = w * w + m * m - r2_dot(r2_sub(a, d), r2_sub(b, d)) - r2_dot(r2_sub(a, d), r2_sub(b, d))
        stewart_expand_ac(a, c, d, w, n, b_len)
        b_len * b_len = w * w + n * n - r2_dot(r2_sub(a, d), r2_sub(c, d)) - r2_dot(r2_sub(a, d), r2_sub(c, d))
        // substitute the expansions into the products
        b_len * b_len * m = (w * w + n * n - r2_dot(r2_sub(a, d), r2_sub(c, d)) - r2_dot(r2_sub(a, d), r2_sub(c, d))) * m
        c_len * c_len * n = (w * w + m * m - r2_dot(r2_sub(a, d), r2_sub(b, d)) - r2_dot(r2_sub(a, d), r2_sub(b, d))) * n
        b_len * b_len * m + c_len * c_len * n =
            (w * w + n * n - r2_dot(r2_sub(a, d), r2_sub(c, d)) - r2_dot(r2_sub(a, d), r2_sub(c, d))) * m +
            (w * w + m * m - r2_dot(r2_sub(a, d), r2_sub(b, d)) - r2_dot(r2_sub(a, d), r2_sub(b, d))) * n
        // the real-algebra core
        real_stewart_core(w * w, m, n, r2_dot(r2_sub(a, d), r2_sub(b, d)), r2_dot(r2_sub(a, d), r2_sub(c, d)))
        (w * w + n * n - r2_dot(r2_sub(a, d), r2_sub(c, d)) - r2_dot(r2_sub(a, d), r2_sub(c, d))) * m +
            (w * w + m * m - r2_dot(r2_sub(a, d), r2_sub(b, d)) - r2_dot(r2_sub(a, d), r2_sub(b, d))) * n =
            (m + n) * (w * w + m * n)
        b_len * b_len * m + c_len * c_len * n = (m + n) * (w * w + m * n)
    }
}
