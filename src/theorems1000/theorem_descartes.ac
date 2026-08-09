from pair import Pair
from real import Real
from top100 import r2_sub, r2_norm_sq

// Descartes' circle theorem (Descartes' theorem): four mutually tangent
// circles with signed curvatures (bends) `k1`, `k2`, `k3`, `k4` satisfy
//     (k1 + k2 + k3 + k4)^2 = 2 (k1^2 + k2^2 + k3^2 + k4^2).
// The signed curvature of a circle of radius `r` is `k = 1 / r`, positive
// for an ordinary circle and negative for an enclosing circle; the theorem
// holds for any four mutually tangent circles (allowing one to enclose the
// other three), and the equality is Descartes' relation.  It is the
// foundation of Apollonian gaskets.
//
// The statement is recorded with the four circles given by their centers
// `o1`, `o2`, `o3`, `o4` and radii `r1`, `r2`, `r3`, `r4`, mutually
// externally tangent (`|oi - oj|^2 = (ri + rj)^2` for all `i < j`), and the
// bends given by the witnesses `k1`, `k2`, `k3`, `k4` with `ki * ri = 1`.
//
// Proof sketch: given three mutually tangent circles of bends `k1`, `k2`,
// `k3`, the two circles tangent to all three have bends
// `k4 = k1 + k2 + k3 +/- 2 sqrt(k1 k2 + k2 k3 + k3 k1)` (the Soddy
// formula), obtained by translating the tangency equations
// `|oi - oj|^2 = (ri + rj)^2` into `(1/k1, 1/k2, 1/k3, 1/k4)` and solving
// the resulting quadratic: `k4^2 - 2 (k1 + k2 + k3) k4 +
// ((k1+k2+k3)^2 - 4 (k1 k2 + k2 k3 + k3 k1)) = 0`.  Expanding the Soddy
// formula (with the square root eliminated by squaring) yields exactly the
// Descartes relation
// `(k1+k2+k3+k4)^2 = 2 (k1^2 + k2^2 + k3^2 + k4^2)`; conversely the
// relation is a quadratic in `k4` whose roots are the Soddy values.  The
// algebra is a pure polynomial identity in the four bends.
//
// theorem theorems1000_descartes_circle(
//     o1: Pair[Real, Real], o2: Pair[Real, Real],
//     o3: Pair[Real, Real], o4: Pair[Real, Real],
//     r1: Real, r2: Real, r3: Real, r4: Real,
//     k1: Real, k2: Real, k3: Real, k4: Real
// ) {
//     r2_norm_sq(r2_sub(o1, o2)) = (r1 + r2) * (r1 + r2) and
//     r2_norm_sq(r2_sub(o1, o3)) = (r1 + r3) * (r1 + r3) and
//     r2_norm_sq(r2_sub(o1, o4)) = (r1 + r4) * (r1 + r4) and
//     r2_norm_sq(r2_sub(o2, o3)) = (r2 + r3) * (r2 + r3) and
//     r2_norm_sq(r2_sub(o2, o4)) = (r2 + r4) * (r2 + r4) and
//     r2_norm_sq(r2_sub(o3, o4)) = (r3 + r4) * (r3 + r4) and
//     k1 * r1 = Real.1 and
//     k2 * r2 = Real.1 and
//     k3 * r3 = Real.1 and
//     k4 * r4 = Real.1
//     implies
//     (k1 + k2 + k3 + k4) * (k1 + k2 + k3 + k4) =
//         (Real.1 + Real.1) * (k1 * k1 + k2 * k2 + k3 * k3 + k4 * k4)
// }
