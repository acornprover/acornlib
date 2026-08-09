from nat import Nat, mul_assoc, mul_comm, one_pow
from list import partial, partial_scalar_mul, partial_pointwise_eq
from algebra.semigroup import mul_fn
from combinatorics.interface import binomial, binomial_term

numerals Nat

// The multinomial theorem: expanding the n-th power of a three-term sum
//     (a + b + c)^n
// gives, for every triple of exponents (i, j, k) with i + j + k = n, the
// monomial a^i b^j c^k with the trinomial coefficient
//     n! / (i! j! k!).
// The trinomial coefficient is written here as the product of two binomial
// coefficients, `trinomial_coeff(n, i, j) = binom(n, i) * binom(n - i, j)`
// (choose the `i` positions of `a`, then the `j` positions of `b` among the
// remaining `n - i`), which is exactly `n! / (i! j! (n - i - j)!)`.
//
// The theorem is stated with the library's partial sums over `[0, n)`
// (`partial(f, m) = f(0) + ... + f(m - 1)`): the outer sum runs over
// `i = 0, ..., n` and the inner sum over `j = 0, ..., n - i`.
//
// Proof: apply the binomial theorem twice.  First expand `(a + (b + c))^n`
// to get `sum_i binom(n, i) a^i (b + c)^(n - i)`; then expand each
// `(b + c)^(n - i)` to `sum_j binom(n - i, j) b^j c^(n - i - j)` and pull
// the scalar `binom(n, i) a^i` into the inner sum (the library lemma
// `partial_scalar_mul`).  The two expansions are combined with the pointwise
// lemma `partial_pointwise_eq`.

/// The trinomial coefficient `n! / (i! j! (n - i - j)!)`, written as the
/// product of two binomial coefficients: choose the `i` positions of the
/// first variable, then the `j` positions of the second among the rest.
define trinomial_coeff(n: Nat, i: Nat, j: Nat) -> Nat {
    n.binom(i) * (n - i).binom(j)
}

/// The monomial term `trinomial_coeff(n, i, j) * a^i * b^j * c^(n - i - j)`.
define trinomial_term(n: Nat, a: Nat, b: Nat, c: Nat, i: Nat, j: Nat) -> Nat {
    trinomial_coeff(n, i, j) * a.pow(i) * b.pow(j) * c.pow(n - i - j)
}

/// The pulled-in inner summand `binom(n, i) a^i binom(n - i, j) b^j c^(...)`
/// equals the trinomial term pointwise (pure multiplication rearrangement).
theorem trinomial_pointwise(n: Nat, a: Nat, b: Nat, c: Nat, i: Nat, j: Nat) {
    mul_fn(n.binom(i) * a.pow(i), binomial_term(b, c, n - i))(j) =
        trinomial_term(n, a, b, c, i, j)
} by {
    binomial_term(b, c, n - i) = binomial_term(b, c, n - i)
    mul_fn(n.binom(i) * a.pow(i), binomial_term(b, c, n - i))(j) =
        (n.binom(i) * a.pow(i)) * binomial_term(b, c, n - i)(j)
    binomial_term(b, c, n - i)(j) = (n - i).binom(j) * b.pow(j) * c.pow((n - i) - j)
    (n.binom(i) * a.pow(i)) * ((n - i).binom(j) * b.pow(j) * c.pow((n - i) - j)) =
        n.binom(i) * (a.pow(i) * ((n - i).binom(j) * b.pow(j) * c.pow((n - i) - j)))
    mul_assoc(a.pow(i), (n - i).binom(j), b.pow(j) * c.pow((n - i) - j))
    a.pow(i) * (n - i).binom(j) * (b.pow(j) * c.pow((n - i) - j)) =
        a.pow(i) * ((n - i).binom(j) * (b.pow(j) * c.pow((n - i) - j)))
    a.pow(i) * ((n - i).binom(j) * (b.pow(j) * c.pow((n - i) - j))) =
        a.pow(i) * (n - i).binom(j) * (b.pow(j) * c.pow((n - i) - j))
    mul_comm(a.pow(i), (n - i).binom(j))
    a.pow(i) * (n - i).binom(j) = (n - i).binom(j) * a.pow(i)
    a.pow(i) * (n - i).binom(j) * (b.pow(j) * c.pow((n - i) - j)) =
        (n - i).binom(j) * a.pow(i) * (b.pow(j) * c.pow((n - i) - j))
    mul_assoc((n - i).binom(j), a.pow(i), b.pow(j) * c.pow((n - i) - j))
    (n - i).binom(j) * a.pow(i) * (b.pow(j) * c.pow((n - i) - j)) =
        (n - i).binom(j) * (a.pow(i) * (b.pow(j) * c.pow((n - i) - j)))
    a.pow(i) * ((n - i).binom(j) * b.pow(j) * c.pow((n - i) - j)) =
        (n - i).binom(j) * (a.pow(i) * (b.pow(j) * c.pow((n - i) - j)))
    n.binom(i) * (a.pow(i) * ((n - i).binom(j) * b.pow(j) * c.pow((n - i) - j))) =
        n.binom(i) * ((n - i).binom(j) * (a.pow(i) * (b.pow(j) * c.pow((n - i) - j))))
    n.binom(i) * ((n - i).binom(j) * (a.pow(i) * (b.pow(j) * c.pow((n - i) - j)))) =
        n.binom(i) * (n - i).binom(j) * a.pow(i) * (b.pow(j) * c.pow((n - i) - j))
    n.binom(i) * (n - i).binom(j) * a.pow(i) * (b.pow(j) * c.pow((n - i) - j)) =
        n.binom(i) * (n - i).binom(j) * a.pow(i) * b.pow(j) * c.pow((n - i) - j)
    trinomial_coeff(n, i, j) = n.binom(i) * (n - i).binom(j)
    n.binom(i) * (n - i).binom(j) * a.pow(i) * b.pow(j) * c.pow((n - i) - j) =
        trinomial_coeff(n, i, j) * a.pow(i) * b.pow(j) * c.pow((n - i) - j)
    (n - i) - j = n - i - j
    trinomial_coeff(n, i, j) * a.pow(i) * b.pow(j) * c.pow((n - i) - j) =
        trinomial_coeff(n, i, j) * a.pow(i) * b.pow(j) * c.pow(n - i - j)
    mul_fn(n.binom(i) * a.pow(i), binomial_term(b, c, n - i))(j) =
        trinomial_term(n, a, b, c, i, j)
}

/// For `i <= n`, the `i`-th binomial term of `(a + (b + c))^n` equals the
/// inner partial sum of trinomial terms.
theorem trinomial_inner(a: Nat, b: Nat, c: Nat, n: Nat, i: Nat) {
    i <= n implies
        binomial_term(a, b + c, n, i) =
            partial(function(j: Nat) {
                trinomial_term(n, a, b, c, i, j)
            }, n - i + Nat.1)
} by {
    if i <= n {
        binomial_term(a, b + c, n, i) = n.binom(i) * a.pow(i) * (b + c).pow(n - i)
        binomial(b, c, n - i)
        (b + c).pow(n - i) = partial(binomial_term(b, c, n - i), (n - i).suc)
        n.binom(i) * a.pow(i) * (b + c).pow(n - i) =
            n.binom(i) * a.pow(i) * partial(binomial_term(b, c, n - i), (n - i).suc)
        partial_scalar_mul(n.binom(i) * a.pow(i), binomial_term(b, c, n - i), (n - i).suc)
        (n.binom(i) * a.pow(i)) * partial(binomial_term(b, c, n - i), (n - i).suc) =
            partial(mul_fn(n.binom(i) * a.pow(i), binomial_term(b, c, n - i)), (n - i).suc)
        n.binom(i) * a.pow(i) * partial(binomial_term(b, c, n - i), (n - i).suc) =
            partial(mul_fn(n.binom(i) * a.pow(i), binomial_term(b, c, n - i)), (n - i).suc)
        partial_pointwise_eq(mul_fn(n.binom(i) * a.pow(i), binomial_term(b, c, n - i)),
            function(j: Nat) { trinomial_term(n, a, b, c, i, j) }, (n - i).suc)
        partial(mul_fn(n.binom(i) * a.pow(i), binomial_term(b, c, n - i)), (n - i).suc) =
            partial(function(j: Nat) { trinomial_term(n, a, b, c, i, j) }, (n - i).suc)
        (n - i).suc = n - i + Nat.1
        partial(function(j: Nat) { trinomial_term(n, a, b, c, i, j) }, (n - i).suc) =
            partial(function(j: Nat) { trinomial_term(n, a, b, c, i, j) }, n - i + Nat.1)
        binomial_term(a, b + c, n, i) =
            partial(function(j: Nat) { trinomial_term(n, a, b, c, i, j) }, n - i + Nat.1)
    }
}

/// The multinomial theorem: (a + b + c)^n is the sum of trinomial terms
/// over all exponent triples (i, j, n - i - j).
theorem theorems1000_multinomial(a: Nat, b: Nat, c: Nat, n: Nat) {
    (a + b + c).pow(n) = partial(function(i: Nat) {
        partial(function(j: Nat) {
            trinomial_term(n, a, b, c, i, j)
        }, n - i + Nat.1)
    }, n + Nat.1)
} by {
    binomial(a, b + c, n)
    (a + (b + c)).pow(n) = partial(binomial_term(a, b + c, n), n.suc)
    (a + b + c).pow(n) = partial(binomial_term(a, b + c, n), n.suc)
    define f(i: Nat) -> Nat {
        binomial_term(a, b + c, n, i)
    }
    define g(i: Nat) -> Nat {
        partial(function(j: Nat) {
            trinomial_term(n, a, b, c, i, j)
        }, n - i + Nat.1)
    }
    forall(i: Nat) {
        if i < n.suc {
            i <= n
            trinomial_inner(a, b, c, n, i)
            binomial_term(a, b + c, n, i) =
                partial(function(j: Nat) {
                    trinomial_term(n, a, b, c, i, j)
                }, n - i + Nat.1)
            f(i) = g(i)
        }
    }
    partial_pointwise_eq(f, g, n.suc)
    partial(f, n.suc) = partial(g, n.suc)
    partial(binomial_term(a, b + c, n), n.suc) = partial(g, n.suc)
    n.suc = n + Nat.1
    partial(g, n.suc) = partial(g, n + Nat.1)
    (a + b + c).pow(n) = partial(g, n + Nat.1)
    g = function(i: Nat) {
        partial(function(j: Nat) {
            trinomial_term(n, a, b, c, i, j)
        }, n - i + Nat.1)
    }
    partial(g, n + Nat.1) = partial(function(i: Nat) {
        partial(function(j: Nat) {
            trinomial_term(n, a, b, c, i, j)
        }, n - i + Nat.1)
    }, n + Nat.1)
    (a + b + c).pow(n) = partial(function(i: Nat) {
        partial(function(j: Nat) {
            trinomial_term(n, a, b, c, i, j)
        }, n - i + Nat.1)
    }, n + Nat.1)
}

/// The trinomial term at a = b = c = 1 is just the trinomial coefficient.
theorem trinomial_term_unit(n: Nat, i: Nat, j: Nat) {
    trinomial_term(n, Nat.1, Nat.1, Nat.1, i, j) = trinomial_coeff(n, i, j)
} by {
    trinomial_term(n, Nat.1, Nat.1, Nat.1, i, j) =
        trinomial_coeff(n, i, j) * Nat.1.pow(i) * Nat.1.pow(j) * Nat.1.pow(n - i - j)
    one_pow[Nat](i)
    Nat.1.pow(i) = Nat.1
    one_pow[Nat](j)
    Nat.1.pow(j) = Nat.1
    one_pow[Nat](n - i - j)
    Nat.1.pow(n - i - j) = Nat.1
    trinomial_coeff(n, i, j) * Nat.1.pow(i) * Nat.1.pow(j) * Nat.1.pow(n - i - j) =
        trinomial_coeff(n, i, j) * Nat.1 * Nat.1 * Nat.1
    trinomial_coeff(n, i, j) * Nat.1 * Nat.1 * Nat.1 = trinomial_coeff(n, i, j)
    trinomial_term(n, Nat.1, Nat.1, Nat.1, i, j) = trinomial_coeff(n, i, j)
}

/// The sum of the trinomial coefficients over all exponent triples
/// (i, j, n - i - j) is 3^n: the multinomial theorem at a = b = c = 1.
theorem theorems1000_trinomial_row_sum(n: Nat) {
    partial(function(i: Nat) {
        partial(function(j: Nat) {
            trinomial_coeff(n, i, j)
        }, n - i + Nat.1)
    }, n + Nat.1) = Nat.3.pow(n)
} by {
    theorems1000_multinomial(Nat.1, Nat.1, Nat.1, n)
    (Nat.1 + Nat.1 + Nat.1).pow(n) =
        partial(function(i: Nat) {
            partial(function(j: Nat) {
                trinomial_term(n, Nat.1, Nat.1, Nat.1, i, j)
            }, n - i + Nat.1)
        }, n + Nat.1)
    Nat.1 + Nat.1 = Nat.2
    Nat.2 + Nat.1 = Nat.3
    Nat.1 + Nat.1 + Nat.1 = Nat.3
    Nat.3.pow(n) =
        partial(function(i: Nat) {
            partial(function(j: Nat) {
                trinomial_term(n, Nat.1, Nat.1, Nat.1, i, j)
            }, n - i + Nat.1)
        }, n + Nat.1)
    define f(i: Nat) -> Nat {
        partial(function(j: Nat) {
            trinomial_term(n, Nat.1, Nat.1, Nat.1, i, j)
        }, n - i + Nat.1)
    }
    define g(i: Nat) -> Nat {
        partial(function(j: Nat) {
            trinomial_coeff(n, i, j)
        }, n - i + Nat.1)
    }
    forall(i: Nat) {
        if i < n + Nat.1 {
            forall(j: Nat) {
                if j < n - i + Nat.1 {
                    trinomial_term_unit(n, i, j)
                    trinomial_term(n, Nat.1, Nat.1, Nat.1, i, j) = trinomial_coeff(n, i, j)
                }
            }
            partial_pointwise_eq(function(j: Nat) {
                trinomial_term(n, Nat.1, Nat.1, Nat.1, i, j)
            }, function(j: Nat) {
                trinomial_coeff(n, i, j)
            }, n - i + Nat.1)
            partial(function(j: Nat) {
                trinomial_term(n, Nat.1, Nat.1, Nat.1, i, j)
            }, n - i + Nat.1) = partial(function(j: Nat) {
                trinomial_coeff(n, i, j)
            }, n - i + Nat.1)
            f(i) = g(i)
        }
    }
    partial_pointwise_eq(f, g, n + Nat.1)
    partial(f, n + Nat.1) = partial(g, n + Nat.1)
    f = function(i: Nat) {
        partial(function(j: Nat) {
            trinomial_term(n, Nat.1, Nat.1, Nat.1, i, j)
        }, n - i + Nat.1)
    }
    partial(f, n + Nat.1) = partial(function(i: Nat) {
        partial(function(j: Nat) {
            trinomial_term(n, Nat.1, Nat.1, Nat.1, i, j)
        }, n - i + Nat.1)
    }, n + Nat.1)
    Nat.3.pow(n) = partial(f, n + Nat.1)
    Nat.3.pow(n) = partial(g, n + Nat.1)
    g = function(i: Nat) {
        partial(function(j: Nat) {
            trinomial_coeff(n, i, j)
        }, n - i + Nat.1)
    }
    partial(g, n + Nat.1) = partial(function(i: Nat) {
        partial(function(j: Nat) {
            trinomial_coeff(n, i, j)
        }, n - i + Nat.1)
    }, n + Nat.1)
    partial(function(i: Nat) {
        partial(function(j: Nat) {
            trinomial_coeff(n, i, j)
        }, n - i + Nat.1)
    }, n + Nat.1) = Nat.3.pow(n)
}
