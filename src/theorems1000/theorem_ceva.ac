from pair import Pair, pair_ext
from real import Real, mul_left_cancel
from top100 import r2_add, r2_neg, r2_sub
from theorems1000.r2_helpers import r2_smul, r2_cross,
    r2_smul_one, r2_smul_neg, r2_smul_sub_distrib,
    r2_sub_add_pair_distrib, r2_sub_eq_add_neg, r2_sub_reverse_neg,
    r2_add_zero_left, r2_sub_sub_same_base, r2_pair_ext,
    r2_cross_add_left, r2_cross_add_right, r2_cross_smul_left,
    r2_cross_smul_right, r2_cross_swap, r2_cross_self_zero,
    r2_cross_sub_left, r2_cross_sub_right

numerals Real

// Ceva's theorem: in a triangle, the three cevians from the vertices to
// points on the opposite sides are concurrent exactly when the product of the
// three ratios in which they divide the sides equals one.
//
// Points on the sides are written as affine combinations: `d` on `bc` with
// parameter `t1` is `d = (1 - t1) b + t1 c`, and the directed ratio
// `BD/DC = t1/(1 - t1)`; likewise `e = (1 - t2) c + t2 a` on `ca` and
// `f = (1 - t3) a + t3 b` on `ab`.  Concurrency of the cevians `ad`, `be`,
// `cf` at the point `p` is the three collinearity conditions
// `cross(p - a, d - a) = 0`, `cross(p - b, e - b) = 0` and
// `cross(p - c, f - c) = 0`.  Ceva's theorem then reads
//     t1 t2 t3 = (1 - t1)(1 - t2)(1 - t3),
// which is the cross-multiplied form of `(BD/DC)(CE/EA)(AF/FB) = 1`.
//
// Proof (the direction "concurrent implies the product equals one"): write
// `u = b - a`, `v = c - a`, `W = cross(u, v)`, `X = cross(p - a, u)` and
// `Y = cross(p - a, v)`.  From the affine combinations,
//     d - a = (1 - t1) u + t1 v,
//     e - b = (1 - t2) v - u,
//     f - c = t3 u - v,
// and `p - b = (p - a) - u`, `p - c = (p - a) - v`, so expanding the three
// cross products with the bilinearity lemmas gives
//     (1 - t1) X + t1 Y = 0,
//     (1 - t2) Y - X - (1 - t2) W = 0,
//     t3 X - Y + t3 W = 0.
// The second and third equations give `X = (1 - t2)(Y - W)` and
// `Y = t3 (X + W)`; substituting each into the other eliminates the two
// unknowns and leaves
//     X (1 - (1 - t2) t3) = -(1 - t2)(1 - t3) W,
//     Y (1 - (1 - t2) t3) = t2 t3 W.
// Multiplying the first equation by `1 - (1 - t2) t3` and substituting these
// two identities gives `W (t1 t2 t3 - (1 - t1)(1 - t2)(1 - t3)) = 0`, and the
// non-degeneracy hypothesis `W != 0` allows cancelling `W`, so
// `t1 t2 t3 = (1 - t1)(1 - t2)(1 - t3)`.  (The converse direction, in which
// the product condition implies concurrency, holds for a nondegenerate
// configuration and is not stated here.)

// ---------------------------------------------------------------------------
// Small vector-algebra helpers.
// ---------------------------------------------------------------------------

/// Scalar multiplication acts componentwise.
theorem ceva_smul_expand(k: Real, a: Pair[Real, Real]) {
    r2_smul(k, a) = Pair.new(k * a.first, k * a.second)
}

/// Scalar multiplication distributes over scalar addition: `k1 a + k2 a =
/// (k1 + k2) a`.
theorem ceva_smul_add_scalar(k1: Real, k2: Real, a: Pair[Real, Real]) {
    r2_add(r2_smul(k1, a), r2_smul(k2, a)) = r2_smul(k1 + k2, a)
} by {
    ceva_smul_expand(k1, a)
    r2_smul(k1, a) = Pair.new(k1 * a.first, k1 * a.second)
    ceva_smul_expand(k2, a)
    r2_smul(k2, a) = Pair.new(k2 * a.first, k2 * a.second)
    ceva_smul_expand(k1 + k2, a)
    r2_smul(k1 + k2, a) = Pair.new((k1 + k2) * a.first, (k1 + k2) * a.second)
    r2_add(r2_smul(k1, a), r2_smul(k2, a)) =
        r2_add(
            Pair.new(k1 * a.first, k1 * a.second),
            Pair.new(k2 * a.first, k2 * a.second))
    r2_add(
        Pair.new(k1 * a.first, k1 * a.second),
        Pair.new(k2 * a.first, k2 * a.second)) =
        Pair.new(k1 * a.first + k2 * a.first, k1 * a.second + k2 * a.second)
    r2_add(r2_smul(k1, a), r2_smul(k2, a)) =
        Pair.new(k1 * a.first + k2 * a.first, k1 * a.second + k2 * a.second)
    k1 * a.first + k2 * a.first = (k1 + k2) * a.first
    k1 * a.second + k2 * a.second = (k1 + k2) * a.second
    r2_pair_ext(
        r2_add(r2_smul(k1, a), r2_smul(k2, a)),
        Pair.new((k1 + k2) * a.first, (k1 + k2) * a.second))
    r2_add(r2_smul(k1, a), r2_smul(k2, a)) =
        Pair.new((k1 + k2) * a.first, (k1 + k2) * a.second)
    r2_add(r2_smul(k1, a), r2_smul(k2, a)) = r2_smul(k1 + k2, a)
}

/// Scaling a point by zero gives the origin.
theorem ceva_smul_zero(a: Pair[Real, Real]) {
    r2_smul(Real.0, a) = Pair.new(Real.0, Real.0)
} by {
    let lhs = r2_smul(Real.0, a)
    lhs.first = Real.0 * a.first
    Real.0 * a.first = Real.0
    lhs.second = Real.0 * a.second
    Real.0 * a.second = Real.0
    pair_ext(lhs, Pair.new(Real.0, Real.0))
}

/// Subtracting a sum successively: (x - y) - z = x - (y + z).
theorem ceva_sub_sum(x: Pair[Real, Real], y: Pair[Real, Real], z: Pair[Real, Real]) {
    r2_sub(r2_sub(x, y), z) = r2_sub(x, r2_add(y, z))
} by {
    let lhs = r2_sub(r2_sub(x, y), z)
    let rhs = r2_sub(x, r2_add(y, z))
    lhs.first = r2_sub(x, y).first - z.first
    r2_sub(x, y).first = x.first - y.first
    lhs.first = (x.first - y.first) - z.first
    rhs.first = x.first - r2_add(y, z).first
    r2_add(y, z).first = y.first + z.first
    rhs.first = x.first - (y.first + z.first)
    (x.first - y.first) - z.first = x.first - (y.first + z.first)
    lhs.first = rhs.first
    lhs.second = r2_sub(x, y).second - z.second
    r2_sub(x, y).second = x.second - y.second
    lhs.second = (x.second - y.second) - z.second
    rhs.second = x.second - r2_add(y, z).second
    r2_add(y, z).second = y.second + z.second
    rhs.second = x.second - (y.second + z.second)
    (x.second - y.second) - z.second = x.second - (y.second + z.second)
    lhs.second = rhs.second
    pair_ext(lhs, rhs)
}

/// Adding a negation is subtracting: x + (-y) = x - y.
theorem ceva_add_neg_sub(x: Pair[Real, Real], y: Pair[Real, Real]) {
    r2_add(x, r2_neg(y)) = r2_sub(x, y)
} by {
    let lhs = r2_add(x, r2_neg(y))
    let rhs = r2_sub(x, y)
    lhs.first = x.first + r2_neg(y).first
    r2_neg(y).first = -y.first
    lhs.first = x.first + -y.first
    rhs.first = x.first - y.first
    x.first + -y.first = x.first - y.first
    lhs.first = rhs.first
    lhs.second = x.second + r2_neg(y).second
    r2_neg(y).second = -y.second
    lhs.second = x.second + -y.second
    rhs.second = x.second - y.second
    x.second + -y.second = x.second - y.second
    lhs.second = rhs.second
    pair_ext(lhs, rhs)
}

/// A negation added to a difference reassociates: -x + (y - z) = (y - x) - z.
theorem ceva_neg_add_sub(x: Pair[Real, Real], y: Pair[Real, Real], z: Pair[Real, Real]) {
    r2_add(r2_neg(x), r2_sub(y, z)) = r2_sub(r2_sub(y, x), z)
} by {
    let lhs = r2_add(r2_neg(x), r2_sub(y, z))
    let rhs = r2_sub(r2_sub(y, x), z)
    lhs.first = r2_neg(x).first + r2_sub(y, z).first
    r2_neg(x).first = -x.first
    r2_sub(y, z).first = y.first - z.first
    lhs.first = -x.first + (y.first - z.first)
    rhs.first = r2_sub(y, x).first - z.first
    r2_sub(y, x).first = y.first - x.first
    rhs.first = (y.first - x.first) - z.first
    -x.first + (y.first - z.first) = (y.first - x.first) - z.first
    lhs.first = rhs.first
    lhs.second = r2_neg(x).second + r2_sub(y, z).second
    r2_neg(x).second = -x.second
    r2_sub(y, z).second = y.second - z.second
    lhs.second = -x.second + (y.second - z.second)
    rhs.second = r2_sub(y, x).second - z.second
    r2_sub(y, x).second = y.second - x.second
    rhs.second = (y.second - x.second) - z.second
    -x.second + (y.second - z.second) = (y.second - x.second) - z.second
    lhs.second = rhs.second
    pair_ext(lhs, rhs)
}

// ---------------------------------------------------------------------------
// The affine decompositions of the side points.
// ---------------------------------------------------------------------------

/// With `d = (1 - t1) b + t1 c` on `bc`, the displacement `d - a` is
/// `(1 - t1)(b - a) + t1 (c - a)`.
theorem ceva_d_sub_a(
    a: Pair[Real, Real], b: Pair[Real, Real], c: Pair[Real, Real],
    d: Pair[Real, Real], t1: Real
) {
    r2_add(r2_smul(Real.1 - t1, b), r2_smul(t1, c)) = d
    implies
    r2_sub(d, a) = r2_add(r2_smul(Real.1 - t1, r2_sub(b, a)), r2_smul(t1, r2_sub(c, a)))
} by {
    if r2_add(r2_smul(Real.1 - t1, b), r2_smul(t1, c)) = d {
        r2_smul_sub_distrib(Real.1 - t1, b, a)
        r2_smul_sub_distrib(t1, c, a)
        r2_add(r2_smul(Real.1 - t1, r2_sub(b, a)), r2_smul(t1, r2_sub(c, a))) =
            r2_add(
                r2_sub(r2_smul(Real.1 - t1, b), r2_smul(Real.1 - t1, a)),
                r2_sub(r2_smul(t1, c), r2_smul(t1, a)))
        r2_sub_add_pair_distrib(
            r2_smul(Real.1 - t1, b), r2_smul(t1, c),
            r2_smul(Real.1 - t1, a), r2_smul(t1, a))
        r2_add(
            r2_sub(r2_smul(Real.1 - t1, b), r2_smul(Real.1 - t1, a)),
            r2_sub(r2_smul(t1, c), r2_smul(t1, a))) =
            r2_sub(
                r2_add(r2_smul(Real.1 - t1, b), r2_smul(t1, c)),
                r2_add(r2_smul(Real.1 - t1, a), r2_smul(t1, a)))
        r2_add(r2_smul(Real.1 - t1, b), r2_smul(t1, c)) = d
        r2_add(r2_smul(Real.1 - t1, r2_sub(b, a)), r2_smul(t1, r2_sub(c, a))) =
            r2_sub(d, r2_add(r2_smul(Real.1 - t1, a), r2_smul(t1, a)))
        ceva_smul_add_scalar(Real.1 - t1, t1, a)
        r2_add(r2_smul(Real.1 - t1, a), r2_smul(t1, a)) = r2_smul(Real.1 - t1 + t1, a)
        Real.1 - t1 + t1 = Real.1
        r2_smul(Real.1 - t1 + t1, a) = r2_smul(Real.1, a)
        r2_smul_one(a)
        r2_smul(Real.1, a) = a
        r2_smul(Real.1 - t1 + t1, a) = a
        r2_add(r2_smul(Real.1 - t1, a), r2_smul(t1, a)) = a
        r2_sub(d, r2_add(r2_smul(Real.1 - t1, a), r2_smul(t1, a))) = r2_sub(d, a)
        r2_add(r2_smul(Real.1 - t1, r2_sub(b, a)), r2_smul(t1, r2_sub(c, a))) = r2_sub(d, a)
        r2_sub(d, a) = r2_add(r2_smul(Real.1 - t1, r2_sub(b, a)), r2_smul(t1, r2_sub(c, a)))
    }
}

/// With `e = (1 - t2) c + t2 a` on `ca`, the displacement `e - b` is
/// `(1 - t2)(c - a) - (b - a)`.
theorem ceva_e_sub_b(
    a: Pair[Real, Real], b: Pair[Real, Real], c: Pair[Real, Real],
    e: Pair[Real, Real], t2: Real
) {
    r2_add(r2_smul(Real.1 - t2, c), r2_smul(t2, a)) = e
    implies
    r2_sub(e, b) = r2_sub(r2_smul(Real.1 - t2, r2_sub(c, a)), r2_sub(b, a))
} by {
    if r2_add(r2_smul(Real.1 - t2, c), r2_smul(t2, a)) = e {
        // b = (1 - t2) b + t2 b
        ceva_smul_add_scalar(Real.1 - t2, t2, b)
        r2_add(r2_smul(Real.1 - t2, b), r2_smul(t2, b)) = r2_smul(Real.1 - t2 + t2, b)
        Real.1 - t2 + t2 = Real.1
        r2_smul(Real.1 - t2 + t2, b) = r2_smul(Real.1, b)
        r2_smul_one(b)
        r2_smul(Real.1, b) = b
        r2_add(r2_smul(Real.1 - t2, b), r2_smul(t2, b)) = b
        // e - b = [(1-t2)c + t2a] - [(1-t2)b + t2b]
        r2_sub(e, b) = r2_sub(
            r2_add(r2_smul(Real.1 - t2, c), r2_smul(t2, a)),
            r2_add(r2_smul(Real.1 - t2, b), r2_smul(t2, b)))
        r2_sub_add_pair_distrib(
            r2_smul(Real.1 - t2, c), r2_smul(t2, a),
            r2_smul(Real.1 - t2, b), r2_smul(t2, b))
        r2_sub(e, b) = r2_add(
            r2_sub(r2_smul(Real.1 - t2, c), r2_smul(Real.1 - t2, b)),
            r2_sub(r2_smul(t2, a), r2_smul(t2, b)))
        r2_smul_sub_distrib(Real.1 - t2, c, b)
        r2_smul_sub_distrib(t2, a, b)
        r2_sub(e, b) = r2_add(r2_smul(Real.1 - t2, r2_sub(c, b)), r2_smul(t2, r2_sub(a, b)))
        // c - b = (c - a) - (b - a)
        r2_sub_sub_same_base(a, b, c)
        r2_sub(r2_sub(c, a), r2_sub(b, a)) = r2_sub(c, b)
        r2_sub(c, b) = r2_sub(r2_sub(c, a), r2_sub(b, a))
        r2_smul(Real.1 - t2, r2_sub(c, b)) =
            r2_smul(Real.1 - t2, r2_sub(r2_sub(c, a), r2_sub(b, a)))
        // a - b = -(b - a)
        r2_sub_reverse_neg(a, b)
        r2_sub(a, b) = r2_neg(r2_sub(b, a))
        r2_smul(t2, r2_sub(a, b)) = r2_smul(t2, r2_neg(r2_sub(b, a)))
        r2_smul_neg(t2, r2_sub(b, a))
        r2_smul(t2, r2_neg(r2_sub(b, a))) = r2_neg(r2_smul(t2, r2_sub(b, a)))
        r2_smul(t2, r2_sub(a, b)) = r2_neg(r2_smul(t2, r2_sub(b, a)))
        r2_sub(e, b) = r2_add(
            r2_smul(Real.1 - t2, r2_sub(r2_sub(c, a), r2_sub(b, a))),
            r2_neg(r2_smul(t2, r2_sub(b, a))))
        ceva_add_neg_sub(
            r2_smul(Real.1 - t2, r2_sub(r2_sub(c, a), r2_sub(b, a))),
            r2_smul(t2, r2_sub(b, a)))
        r2_add(
            r2_smul(Real.1 - t2, r2_sub(r2_sub(c, a), r2_sub(b, a))),
            r2_neg(r2_smul(t2, r2_sub(b, a)))) =
            r2_sub(
                r2_smul(Real.1 - t2, r2_sub(r2_sub(c, a), r2_sub(b, a))),
                r2_smul(t2, r2_sub(b, a)))
        r2_sub(e, b) = r2_sub(
            r2_smul(Real.1 - t2, r2_sub(r2_sub(c, a), r2_sub(b, a))),
            r2_smul(t2, r2_sub(b, a)))
        r2_smul_sub_distrib(Real.1 - t2, r2_sub(c, a), r2_sub(b, a))
        r2_smul(Real.1 - t2, r2_sub(r2_sub(c, a), r2_sub(b, a))) =
            r2_sub(
                r2_smul(Real.1 - t2, r2_sub(c, a)),
                r2_smul(Real.1 - t2, r2_sub(b, a)))
        r2_sub(e, b) = r2_sub(
            r2_sub(
                r2_smul(Real.1 - t2, r2_sub(c, a)),
                r2_smul(Real.1 - t2, r2_sub(b, a))),
            r2_smul(t2, r2_sub(b, a)))
        ceva_sub_sum(
            r2_smul(Real.1 - t2, r2_sub(c, a)),
            r2_smul(Real.1 - t2, r2_sub(b, a)),
            r2_smul(t2, r2_sub(b, a)))
        r2_sub(
            r2_sub(
                r2_smul(Real.1 - t2, r2_sub(c, a)),
                r2_smul(Real.1 - t2, r2_sub(b, a))),
            r2_smul(t2, r2_sub(b, a))) =
            r2_sub(
                r2_smul(Real.1 - t2, r2_sub(c, a)),
                r2_add(
                    r2_smul(Real.1 - t2, r2_sub(b, a)),
                    r2_smul(t2, r2_sub(b, a))))
        r2_sub(e, b) = r2_sub(
            r2_smul(Real.1 - t2, r2_sub(c, a)),
            r2_add(
                r2_smul(Real.1 - t2, r2_sub(b, a)),
                r2_smul(t2, r2_sub(b, a))))
        ceva_smul_add_scalar(Real.1 - t2, t2, r2_sub(b, a))
        r2_add(
            r2_smul(Real.1 - t2, r2_sub(b, a)),
            r2_smul(t2, r2_sub(b, a))) = r2_smul(Real.1 - t2 + t2, r2_sub(b, a))
        Real.1 - t2 + t2 = Real.1
        r2_smul(Real.1 - t2 + t2, r2_sub(b, a)) = r2_smul(Real.1, r2_sub(b, a))
        r2_smul_one(r2_sub(b, a))
        r2_smul(Real.1, r2_sub(b, a)) = r2_sub(b, a)
        r2_add(
            r2_smul(Real.1 - t2, r2_sub(b, a)),
            r2_smul(t2, r2_sub(b, a))) = r2_sub(b, a)
        r2_sub(e, b) = r2_sub(r2_smul(Real.1 - t2, r2_sub(c, a)), r2_sub(b, a))
    }
}

/// With `f = (1 - t3) a + t3 b` on `ab`, the displacement `f - c` is
/// `t3 (b - a) - (c - a)`.
theorem ceva_f_sub_c(
    a: Pair[Real, Real], b: Pair[Real, Real], c: Pair[Real, Real],
    f: Pair[Real, Real], t3: Real
) {
    r2_add(r2_smul(Real.1 - t3, a), r2_smul(t3, b)) = f
    implies
    r2_sub(f, c) = r2_sub(r2_smul(t3, r2_sub(b, a)), r2_sub(c, a))
} by {
    if r2_add(r2_smul(Real.1 - t3, a), r2_smul(t3, b)) = f {
        // c = (1 - t3) c + t3 c
        ceva_smul_add_scalar(Real.1 - t3, t3, c)
        r2_add(r2_smul(Real.1 - t3, c), r2_smul(t3, c)) = r2_smul(Real.1 - t3 + t3, c)
        Real.1 - t3 + t3 = Real.1
        r2_smul(Real.1 - t3 + t3, c) = r2_smul(Real.1, c)
        r2_smul_one(c)
        r2_smul(Real.1, c) = c
        r2_add(r2_smul(Real.1 - t3, c), r2_smul(t3, c)) = c
        // f - c = [(1-t3)a + t3b] - [(1-t3)c + t3c]
        r2_sub(f, c) = r2_sub(
            r2_add(r2_smul(Real.1 - t3, a), r2_smul(t3, b)),
            r2_add(r2_smul(Real.1 - t3, c), r2_smul(t3, c)))
        r2_sub_add_pair_distrib(
            r2_smul(Real.1 - t3, a), r2_smul(t3, b),
            r2_smul(Real.1 - t3, c), r2_smul(t3, c))
        r2_sub(f, c) = r2_add(
            r2_sub(r2_smul(Real.1 - t3, a), r2_smul(Real.1 - t3, c)),
            r2_sub(r2_smul(t3, b), r2_smul(t3, c)))
        r2_smul_sub_distrib(Real.1 - t3, a, c)
        r2_smul_sub_distrib(t3, b, c)
        r2_sub(f, c) = r2_add(r2_smul(Real.1 - t3, r2_sub(a, c)), r2_smul(t3, r2_sub(b, c)))
        // a - c = -(c - a)
        r2_sub_reverse_neg(a, c)
        r2_sub(a, c) = r2_neg(r2_sub(c, a))
        r2_smul(Real.1 - t3, r2_sub(a, c)) = r2_smul(Real.1 - t3, r2_neg(r2_sub(c, a)))
        r2_smul_neg(Real.1 - t3, r2_sub(c, a))
        r2_smul(Real.1 - t3, r2_neg(r2_sub(c, a))) =
            r2_neg(r2_smul(Real.1 - t3, r2_sub(c, a)))
        r2_smul(Real.1 - t3, r2_sub(a, c)) = r2_neg(r2_smul(Real.1 - t3, r2_sub(c, a)))
        // b - c = (b - a) - (c - a)
        r2_sub_sub_same_base(a, c, b)
        r2_sub(r2_sub(b, a), r2_sub(c, a)) = r2_sub(b, c)
        r2_sub(b, c) = r2_sub(r2_sub(b, a), r2_sub(c, a))
        r2_smul(t3, r2_sub(b, c)) = r2_smul(t3, r2_sub(r2_sub(b, a), r2_sub(c, a)))
        r2_smul_sub_distrib(t3, r2_sub(b, a), r2_sub(c, a))
        r2_smul(t3, r2_sub(r2_sub(b, a), r2_sub(c, a))) =
            r2_sub(r2_smul(t3, r2_sub(b, a)), r2_smul(t3, r2_sub(c, a)))
        r2_smul(t3, r2_sub(b, c)) =
            r2_sub(r2_smul(t3, r2_sub(b, a)), r2_smul(t3, r2_sub(c, a)))
        r2_sub(f, c) = r2_add(
            r2_neg(r2_smul(Real.1 - t3, r2_sub(c, a))),
            r2_sub(r2_smul(t3, r2_sub(b, a)), r2_smul(t3, r2_sub(c, a))))
        ceva_neg_add_sub(
            r2_smul(Real.1 - t3, r2_sub(c, a)),
            r2_smul(t3, r2_sub(b, a)),
            r2_smul(t3, r2_sub(c, a)))
        r2_add(
            r2_neg(r2_smul(Real.1 - t3, r2_sub(c, a))),
            r2_sub(r2_smul(t3, r2_sub(b, a)), r2_smul(t3, r2_sub(c, a)))) =
            r2_sub(
                r2_sub(r2_smul(t3, r2_sub(b, a)), r2_smul(Real.1 - t3, r2_sub(c, a))),
                r2_smul(t3, r2_sub(c, a)))
        r2_sub(f, c) = r2_sub(
            r2_sub(r2_smul(t3, r2_sub(b, a)), r2_smul(Real.1 - t3, r2_sub(c, a))),
            r2_smul(t3, r2_sub(c, a)))
        ceva_sub_sum(
            r2_smul(t3, r2_sub(b, a)),
            r2_smul(Real.1 - t3, r2_sub(c, a)),
            r2_smul(t3, r2_sub(c, a)))
        r2_sub(
            r2_sub(r2_smul(t3, r2_sub(b, a)), r2_smul(Real.1 - t3, r2_sub(c, a))),
            r2_smul(t3, r2_sub(c, a))) =
            r2_sub(
                r2_smul(t3, r2_sub(b, a)),
                r2_add(
                    r2_smul(Real.1 - t3, r2_sub(c, a)),
                    r2_smul(t3, r2_sub(c, a))))
        r2_sub(f, c) = r2_sub(
            r2_smul(t3, r2_sub(b, a)),
            r2_add(
                r2_smul(Real.1 - t3, r2_sub(c, a)),
                r2_smul(t3, r2_sub(c, a))))
        ceva_smul_add_scalar(Real.1 - t3, t3, r2_sub(c, a))
        r2_add(
            r2_smul(Real.1 - t3, r2_sub(c, a)),
            r2_smul(t3, r2_sub(c, a))) = r2_smul(Real.1 - t3 + t3, r2_sub(c, a))
        Real.1 - t3 + t3 = Real.1
        r2_smul(Real.1 - t3 + t3, r2_sub(c, a)) = r2_smul(Real.1, r2_sub(c, a))
        r2_smul_one(r2_sub(c, a))
        r2_smul(Real.1, r2_sub(c, a)) = r2_sub(c, a)
        r2_add(
            r2_smul(Real.1 - t3, r2_sub(c, a)),
            r2_smul(t3, r2_sub(c, a))) = r2_sub(c, a)
        r2_sub(f, c) = r2_sub(r2_smul(t3, r2_sub(b, a)), r2_sub(c, a))
    }
}

// ---------------------------------------------------------------------------
// The cross-product expansions.
// ---------------------------------------------------------------------------

/// The first collinearity expands: with `X = cross(p - a, b - a)` and
/// `Y = cross(p - a, c - a)`, `cross(p - a, (1 - t1)(b - a) + t1 (c - a)) =
/// (1 - t1) X + t1 Y`.
theorem ceva_cross_pd(
    p: Pair[Real, Real], a: Pair[Real, Real], b: Pair[Real, Real],
    c: Pair[Real, Real], t1: Real
) {
    r2_cross(
        r2_sub(p, a),
        r2_add(r2_smul(Real.1 - t1, r2_sub(b, a)), r2_smul(t1, r2_sub(c, a)))
    ) =
    (Real.1 - t1) * r2_cross(r2_sub(p, a), r2_sub(b, a)) +
        t1 * r2_cross(r2_sub(p, a), r2_sub(c, a))
} by {
    r2_cross_add_right(
        r2_sub(p, a),
        r2_smul(Real.1 - t1, r2_sub(b, a)),
        r2_smul(t1, r2_sub(c, a)))
    r2_cross(
        r2_sub(p, a),
        r2_add(r2_smul(Real.1 - t1, r2_sub(b, a)), r2_smul(t1, r2_sub(c, a)))
    ) =
    r2_cross(r2_sub(p, a), r2_smul(Real.1 - t1, r2_sub(b, a))) +
        r2_cross(r2_sub(p, a), r2_smul(t1, r2_sub(c, a)))
    r2_cross_smul_right(Real.1 - t1, r2_sub(b, a), r2_sub(p, a))
    r2_cross(r2_sub(p, a), r2_smul(Real.1 - t1, r2_sub(b, a))) =
        (Real.1 - t1) * r2_cross(r2_sub(p, a), r2_sub(b, a))
    r2_cross_smul_right(t1, r2_sub(c, a), r2_sub(p, a))
    r2_cross(r2_sub(p, a), r2_smul(t1, r2_sub(c, a))) =
        t1 * r2_cross(r2_sub(p, a), r2_sub(c, a))
    r2_cross(
        r2_sub(p, a),
        r2_add(r2_smul(Real.1 - t1, r2_sub(b, a)), r2_smul(t1, r2_sub(c, a)))
    ) =
    (Real.1 - t1) * r2_cross(r2_sub(p, a), r2_sub(b, a)) +
        t1 * r2_cross(r2_sub(p, a), r2_sub(c, a))
}

/// The second collinearity expands: with `X = cross(p - a, b - a)`,
/// `Y = cross(p - a, c - a)` and `W = cross(b - a, c - a)`,
/// `cross((p - a) - (b - a), (1 - t2)(c - a) - (b - a)) =
/// (1 - t2) Y - X - (1 - t2) W`.
theorem ceva_cross_pb_eb(
    p: Pair[Real, Real], a: Pair[Real, Real], b: Pair[Real, Real],
    c: Pair[Real, Real], t2: Real
) {
    r2_cross(
        r2_sub(r2_sub(p, a), r2_sub(b, a)),
        r2_sub(r2_smul(Real.1 - t2, r2_sub(c, a)), r2_sub(b, a))
    ) =
    (Real.1 - t2) * r2_cross(r2_sub(p, a), r2_sub(c, a)) -
        r2_cross(r2_sub(p, a), r2_sub(b, a)) -
        (Real.1 - t2) * r2_cross(r2_sub(b, a), r2_sub(c, a))
} by {
    r2_cross_sub_left(
        r2_sub(p, a), r2_sub(b, a),
        r2_sub(r2_smul(Real.1 - t2, r2_sub(c, a)), r2_sub(b, a)))
    r2_cross(
        r2_sub(r2_sub(p, a), r2_sub(b, a)),
        r2_sub(r2_smul(Real.1 - t2, r2_sub(c, a)), r2_sub(b, a))
    ) =
    r2_cross(
        r2_sub(p, a),
        r2_sub(r2_smul(Real.1 - t2, r2_sub(c, a)), r2_sub(b, a))
    ) -
    r2_cross(
        r2_sub(b, a),
        r2_sub(r2_smul(Real.1 - t2, r2_sub(c, a)), r2_sub(b, a))
    )
    r2_cross_sub_right(
        r2_sub(p, a), r2_smul(Real.1 - t2, r2_sub(c, a)), r2_sub(b, a))
    r2_cross(
        r2_sub(p, a),
        r2_sub(r2_smul(Real.1 - t2, r2_sub(c, a)), r2_sub(b, a))
    ) =
    r2_cross(r2_sub(p, a), r2_smul(Real.1 - t2, r2_sub(c, a))) -
        r2_cross(r2_sub(p, a), r2_sub(b, a))
    r2_cross_smul_right(Real.1 - t2, r2_sub(c, a), r2_sub(p, a))
    r2_cross(r2_sub(p, a), r2_smul(Real.1 - t2, r2_sub(c, a))) =
        (Real.1 - t2) * r2_cross(r2_sub(p, a), r2_sub(c, a))
    r2_cross(
        r2_sub(p, a),
        r2_sub(r2_smul(Real.1 - t2, r2_sub(c, a)), r2_sub(b, a))
    ) =
    (Real.1 - t2) * r2_cross(r2_sub(p, a), r2_sub(c, a)) -
        r2_cross(r2_sub(p, a), r2_sub(b, a))
    r2_cross_sub_right(
        r2_sub(b, a), r2_smul(Real.1 - t2, r2_sub(c, a)), r2_sub(b, a))
    r2_cross(
        r2_sub(b, a),
        r2_sub(r2_smul(Real.1 - t2, r2_sub(c, a)), r2_sub(b, a))
    ) =
    r2_cross(r2_sub(b, a), r2_smul(Real.1 - t2, r2_sub(c, a))) -
        r2_cross(r2_sub(b, a), r2_sub(b, a))
    r2_cross_smul_right(Real.1 - t2, r2_sub(c, a), r2_sub(b, a))
    r2_cross(r2_sub(b, a), r2_smul(Real.1 - t2, r2_sub(c, a))) =
        (Real.1 - t2) * r2_cross(r2_sub(b, a), r2_sub(c, a))
    r2_cross_self_zero(r2_sub(b, a))
    r2_cross(r2_sub(b, a), r2_sub(b, a)) = Real.0
    r2_cross(
        r2_sub(b, a),
        r2_sub(r2_smul(Real.1 - t2, r2_sub(c, a)), r2_sub(b, a))
    ) =
    (Real.1 - t2) * r2_cross(r2_sub(b, a), r2_sub(c, a)) - Real.0
    (Real.1 - t2) * r2_cross(r2_sub(b, a), r2_sub(c, a)) - Real.0 =
        (Real.1 - t2) * r2_cross(r2_sub(b, a), r2_sub(c, a))
    r2_cross(
        r2_sub(b, a),
        r2_sub(r2_smul(Real.1 - t2, r2_sub(c, a)), r2_sub(b, a))
    ) =
    (Real.1 - t2) * r2_cross(r2_sub(b, a), r2_sub(c, a))
    r2_cross(
        r2_sub(r2_sub(p, a), r2_sub(b, a)),
        r2_sub(r2_smul(Real.1 - t2, r2_sub(c, a)), r2_sub(b, a))
    ) =
    ((Real.1 - t2) * r2_cross(r2_sub(p, a), r2_sub(c, a)) -
        r2_cross(r2_sub(p, a), r2_sub(b, a))) -
        (Real.1 - t2) * r2_cross(r2_sub(b, a), r2_sub(c, a))
    ((Real.1 - t2) * r2_cross(r2_sub(p, a), r2_sub(c, a)) -
        r2_cross(r2_sub(p, a), r2_sub(b, a))) -
        (Real.1 - t2) * r2_cross(r2_sub(b, a), r2_sub(c, a)) =
        (Real.1 - t2) * r2_cross(r2_sub(p, a), r2_sub(c, a)) -
        r2_cross(r2_sub(p, a), r2_sub(b, a)) -
        (Real.1 - t2) * r2_cross(r2_sub(b, a), r2_sub(c, a))
    r2_cross(
        r2_sub(r2_sub(p, a), r2_sub(b, a)),
        r2_sub(r2_smul(Real.1 - t2, r2_sub(c, a)), r2_sub(b, a))
    ) =
    (Real.1 - t2) * r2_cross(r2_sub(p, a), r2_sub(c, a)) -
        r2_cross(r2_sub(p, a), r2_sub(b, a)) -
        (Real.1 - t2) * r2_cross(r2_sub(b, a), r2_sub(c, a))
}

/// The third collinearity expands: `cross((p - a) - (c - a), t3 (b - a) -
/// (c - a)) = t3 X - Y + t3 W`.
theorem ceva_cross_pc_fc(
    p: Pair[Real, Real], a: Pair[Real, Real], b: Pair[Real, Real],
    c: Pair[Real, Real], t3: Real
) {
    r2_cross(
        r2_sub(r2_sub(p, a), r2_sub(c, a)),
        r2_sub(r2_smul(t3, r2_sub(b, a)), r2_sub(c, a))
    ) =
    t3 * r2_cross(r2_sub(p, a), r2_sub(b, a)) -
        r2_cross(r2_sub(p, a), r2_sub(c, a)) +
        t3 * r2_cross(r2_sub(b, a), r2_sub(c, a))
} by {
    r2_cross_sub_left(
        r2_sub(p, a), r2_sub(c, a),
        r2_sub(r2_smul(t3, r2_sub(b, a)), r2_sub(c, a)))
    r2_cross(
        r2_sub(r2_sub(p, a), r2_sub(c, a)),
        r2_sub(r2_smul(t3, r2_sub(b, a)), r2_sub(c, a))
    ) =
    r2_cross(
        r2_sub(p, a),
        r2_sub(r2_smul(t3, r2_sub(b, a)), r2_sub(c, a))
    ) -
    r2_cross(
        r2_sub(c, a),
        r2_sub(r2_smul(t3, r2_sub(b, a)), r2_sub(c, a))
    )
    r2_cross_sub_right(
        r2_sub(p, a), r2_smul(t3, r2_sub(b, a)), r2_sub(c, a))
    r2_cross(
        r2_sub(p, a),
        r2_sub(r2_smul(t3, r2_sub(b, a)), r2_sub(c, a))
    ) =
    r2_cross(r2_sub(p, a), r2_smul(t3, r2_sub(b, a))) -
        r2_cross(r2_sub(p, a), r2_sub(c, a))
    r2_cross_smul_right(t3, r2_sub(b, a), r2_sub(p, a))
    r2_cross(r2_sub(p, a), r2_smul(t3, r2_sub(b, a))) =
        t3 * r2_cross(r2_sub(p, a), r2_sub(b, a))
    r2_cross(
        r2_sub(p, a),
        r2_sub(r2_smul(t3, r2_sub(b, a)), r2_sub(c, a))
    ) =
    t3 * r2_cross(r2_sub(p, a), r2_sub(b, a)) -
        r2_cross(r2_sub(p, a), r2_sub(c, a))
    r2_cross_sub_right(
        r2_sub(c, a), r2_smul(t3, r2_sub(b, a)), r2_sub(c, a))
    r2_cross(
        r2_sub(c, a),
        r2_sub(r2_smul(t3, r2_sub(b, a)), r2_sub(c, a))
    ) =
    r2_cross(r2_sub(c, a), r2_smul(t3, r2_sub(b, a))) -
        r2_cross(r2_sub(c, a), r2_sub(c, a))
    r2_cross_smul_right(t3, r2_sub(b, a), r2_sub(c, a))
    r2_cross(r2_sub(c, a), r2_smul(t3, r2_sub(b, a))) =
        t3 * r2_cross(r2_sub(c, a), r2_sub(b, a))
    r2_cross_self_zero(r2_sub(c, a))
    r2_cross(r2_sub(c, a), r2_sub(c, a)) = Real.0
    r2_cross(
        r2_sub(c, a),
        r2_sub(r2_smul(t3, r2_sub(b, a)), r2_sub(c, a))
    ) =
    t3 * r2_cross(r2_sub(c, a), r2_sub(b, a)) - Real.0
    t3 * r2_cross(r2_sub(c, a), r2_sub(b, a)) - Real.0 =
        t3 * r2_cross(r2_sub(c, a), r2_sub(b, a))
    r2_cross(
        r2_sub(c, a),
        r2_sub(r2_smul(t3, r2_sub(b, a)), r2_sub(c, a))
    ) =
    t3 * r2_cross(r2_sub(c, a), r2_sub(b, a))
    r2_cross_swap(r2_sub(b, a), r2_sub(c, a))
    r2_cross(r2_sub(b, a), r2_sub(c, a)) = -r2_cross(r2_sub(c, a), r2_sub(b, a))
    r2_cross(r2_sub(c, a), r2_sub(b, a)) = -r2_cross(r2_sub(b, a), r2_sub(c, a))
    t3 * r2_cross(r2_sub(c, a), r2_sub(b, a)) =
        -t3 * r2_cross(r2_sub(b, a), r2_sub(c, a))
    r2_cross(
        r2_sub(c, a),
        r2_sub(r2_smul(t3, r2_sub(b, a)), r2_sub(c, a))
    ) =
    -t3 * r2_cross(r2_sub(b, a), r2_sub(c, a))
    r2_cross(
        r2_sub(r2_sub(p, a), r2_sub(c, a)),
        r2_sub(r2_smul(t3, r2_sub(b, a)), r2_sub(c, a))
    ) =
    (t3 * r2_cross(r2_sub(p, a), r2_sub(b, a)) -
        r2_cross(r2_sub(p, a), r2_sub(c, a))) -
        (-t3 * r2_cross(r2_sub(b, a), r2_sub(c, a)))
    (t3 * r2_cross(r2_sub(p, a), r2_sub(b, a)) -
        r2_cross(r2_sub(p, a), r2_sub(c, a))) -
        (-t3 * r2_cross(r2_sub(b, a), r2_sub(c, a))) =
        t3 * r2_cross(r2_sub(p, a), r2_sub(b, a)) -
        r2_cross(r2_sub(p, a), r2_sub(c, a)) +
        t3 * r2_cross(r2_sub(b, a), r2_sub(c, a))
    r2_cross(
        r2_sub(r2_sub(p, a), r2_sub(c, a)),
        r2_sub(r2_smul(t3, r2_sub(b, a)), r2_sub(c, a))
    ) =
    t3 * r2_cross(r2_sub(p, a), r2_sub(b, a)) -
        r2_cross(r2_sub(p, a), r2_sub(c, a)) +
        t3 * r2_cross(r2_sub(b, a), r2_sub(c, a))
}

// ---------------------------------------------------------------------------
// The scalar eliminations behind Ceva's product identity.
// ---------------------------------------------------------------------------

/// From `(1 - t2) y - x - (1 - t2) w = 0`, `x = (1 - t2)(y - w)`.
theorem ceva_lin1(x: Real, y: Real, w: Real, t2: Real) {
    (Real.1 - t2) * y - x - (Real.1 - t2) * w = Real.0
    implies
    x = (Real.1 - t2) * (y - w)
} by {
    if (Real.1 - t2) * y - x - (Real.1 - t2) * w = Real.0 {
        (Real.1 - t2) * y - x - (Real.1 - t2) * w = Real.0
        (Real.1 - t2) * y - (Real.1 - t2) * w = x
        (Real.1 - t2) * (y - w) = x
        x = (Real.1 - t2) * (y - w)
    }
}

/// From `t3 x - y + t3 w = 0`, `y = t3 (x + w)`.
theorem ceva_lin2(x: Real, y: Real, w: Real, t3: Real) {
    t3 * x - y + t3 * w = Real.0
    implies
    y = t3 * (x + w)
} by {
    if t3 * x - y + t3 * w = Real.0 {
        t3 * x - y + t3 * w = Real.0
        t3 * x + t3 * w = y
        t3 * (x + w) = y
        y = t3 * (x + w)
    }
}

/// Eliminating `y` from `x = (1 - t2)(y - w)` and `y = t3 (x + w)` leaves
/// `x (1 - (1 - t2) t3) = -(1 - t2)(1 - t3) w`.
theorem ceva_elim_x(x: Real, y: Real, w: Real, t2: Real, t3: Real) {
    x = (Real.1 - t2) * (y - w) and y = t3 * (x + w)
    implies
    x * (Real.1 - (Real.1 - t2) * t3) = (Real.1 - t2) * (t3 - Real.1) * w
} by {
    if x = (Real.1 - t2) * (y - w) and y = t3 * (x + w) {
        x = (Real.1 - t2) * (y - w)
        y = t3 * (x + w)
        // substitute y = t3 (x + w) into the equation for x
        (Real.1 - t2) * (y - w) = (Real.1 - t2) * (t3 * (x + w) - w)
        x = (Real.1 - t2) * (t3 * (x + w) - w)
        // t3 (x + w) - w = t3 x + (t3 - 1) w
        t3 * (x + w) = t3 * x + t3 * w
        t3 * (x + w) - w = t3 * x + t3 * w - w
        t3 * w - w = (t3 - Real.1) * w
        t3 * x + t3 * w - w = t3 * x + (t3 - Real.1) * w
        t3 * (x + w) - w = t3 * x + (t3 - Real.1) * w
        (Real.1 - t2) * (t3 * (x + w) - w) = (Real.1 - t2) * (t3 * x + (t3 - Real.1) * w)
        x = (Real.1 - t2) * (t3 * x + (t3 - Real.1) * w)
        // distribute (1 - t2) over the sum
        (Real.1 - t2) * (t3 * x + (t3 - Real.1) * w) =
            (Real.1 - t2) * (t3 * x) + (Real.1 - t2) * ((t3 - Real.1) * w)
        x = (Real.1 - t2) * (t3 * x) + (Real.1 - t2) * ((t3 - Real.1) * w)
        (Real.1 - t2) * (t3 * x) = (Real.1 - t2) * t3 * x
        (Real.1 - t2) * ((t3 - Real.1) * w) = (Real.1 - t2) * (t3 - Real.1) * w
        x = (Real.1 - t2) * t3 * x + (Real.1 - t2) * (t3 - Real.1) * w
        // bring the x terms together
        (Real.1 - t2) * t3 * x + (Real.1 - t2) * (t3 - Real.1) * w -
            (Real.1 - t2) * t3 * x = (Real.1 - t2) * (t3 - Real.1) * w
        x - (Real.1 - t2) * t3 * x = (Real.1 - t2) * (t3 - Real.1) * w
        // factor x
        x - (Real.1 - t2) * t3 * x = (Real.1 - (Real.1 - t2) * t3) * x
        (Real.1 - (Real.1 - t2) * t3) * x = (Real.1 - t2) * (t3 - Real.1) * w
        x * (Real.1 - (Real.1 - t2) * t3) = (Real.1 - t2) * (t3 - Real.1) * w
    }
}

/// Eliminating `x` from `x = (1 - t2)(y - w)` and `y = t3 (x + w)` leaves
/// `y (1 - (1 - t2) t3) = t2 t3 w`.
theorem ceva_elim_y(x: Real, y: Real, w: Real, t2: Real, t3: Real) {
    x = (Real.1 - t2) * (y - w) and y = t3 * (x + w)
    implies
    y * (Real.1 - (Real.1 - t2) * t3) = t2 * t3 * w
} by {
    if x = (Real.1 - t2) * (y - w) and y = t3 * (x + w) {
        x = (Real.1 - t2) * (y - w)
        y = t3 * (x + w)
        // substitute x into the equation for y
        t3 * (x + w) = t3 * ((Real.1 - t2) * (y - w) + w)
        y = t3 * ((Real.1 - t2) * (y - w) + w)
        // (1 - t2)(y - w) + w = (1 - t2) y + t2 w
        (Real.1 - t2) * (y - w) = (Real.1 - t2) * y - (Real.1 - t2) * w
        (Real.1 - t2) * (y - w) + w = (Real.1 - t2) * y - (Real.1 - t2) * w + w
        (Real.1 - t2) * y - (Real.1 - t2) * w + w =
            (Real.1 - t2) * y + (-(Real.1 - t2) * w + w)
        -(Real.1 - t2) * w + w = (Real.1 - (Real.1 - t2)) * w
        Real.1 - (Real.1 - t2) = Real.1 + -(Real.1 - t2)
        -(Real.1 - t2) = -Real.1 + t2
        Real.1 + (-Real.1 + t2) = Real.1 - Real.1 + t2
        Real.1 - (Real.1 - t2) = Real.1 - Real.1 + t2
        Real.1 - Real.1 + t2 = Real.0 + t2
        Real.0 + t2 = t2
        Real.1 - (Real.1 - t2) = t2
        -(Real.1 - t2) * w + w = t2 * w
        (Real.1 - t2) * y + (-(Real.1 - t2) * w + w) = (Real.1 - t2) * y + t2 * w
        (Real.1 - t2) * (y - w) + w = (Real.1 - t2) * y + t2 * w
        y = t3 * ((Real.1 - t2) * y + t2 * w)
        // distribute t3 over the sum
        t3 * ((Real.1 - t2) * y + t2 * w) =
            t3 * ((Real.1 - t2) * y) + t3 * (t2 * w)
        y = t3 * ((Real.1 - t2) * y) + t3 * (t2 * w)
        t3 * ((Real.1 - t2) * y) = t3 * (Real.1 - t2) * y
        t3 * (t2 * w) = t3 * t2 * w
        y = t3 * (Real.1 - t2) * y + t3 * t2 * w
        // bring the y terms together
        t3 * (Real.1 - t2) * y + t3 * t2 * w - t3 * (Real.1 - t2) * y =
            t3 * t2 * w
        y - t3 * (Real.1 - t2) * y = t3 * t2 * w
        // factor y
        y - t3 * (Real.1 - t2) * y = (Real.1 - t3 * (Real.1 - t2)) * y
        (Real.1 - t3 * (Real.1 - t2)) * y = t3 * t2 * w
        y * (Real.1 - t3 * (Real.1 - t2)) = t2 * t3 * w
        t3 * (Real.1 - t2) = (Real.1 - t2) * t3
        Real.1 - t3 * (Real.1 - t2) = Real.1 - (Real.1 - t2) * t3
        y * (Real.1 - (Real.1 - t2) * t3) = t2 * t3 * w
    }
}

/// Multiplying the first collinearity equation by `1 - (1 - t2) t3` and
/// substituting the two elimination identities shows
/// `w (t1 t2 t3 - (1 - t1)(1 - t2)(1 - t3)) = 0`.
theorem ceva_product_zero(t1: Real, t2: Real, t3: Real, x: Real, y: Real, w: Real) {
    (Real.1 - t1) * x + t1 * y = Real.0 and
    x * (Real.1 - (Real.1 - t2) * t3) = (Real.1 - t2) * (t3 - Real.1) * w and
    y * (Real.1 - (Real.1 - t2) * t3) = t2 * t3 * w
    implies
    w * (t1 * t2 * t3 - (Real.1 - t1) * (Real.1 - t2) * (Real.1 - t3)) = Real.0
} by {
    if (Real.1 - t1) * x + t1 * y = Real.0 and
        x * (Real.1 - (Real.1 - t2) * t3) = (Real.1 - t2) * (t3 - Real.1) * w and
        y * (Real.1 - (Real.1 - t2) * t3) = t2 * t3 * w {
        (Real.1 - t1) * x + t1 * y = Real.0
        x * (Real.1 - (Real.1 - t2) * t3) = (Real.1 - t2) * (t3 - Real.1) * w
        y * (Real.1 - (Real.1 - t2) * t3) = t2 * t3 * w
        // multiply the first equation by 1 - (1 - t2) t3
        ((Real.1 - t1) * x + t1 * y) * (Real.1 - (Real.1 - t2) * t3) =
            Real.0 * (Real.1 - (Real.1 - t2) * t3)
        Real.0 * (Real.1 - (Real.1 - t2) * t3) = Real.0
        ((Real.1 - t1) * x + t1 * y) * (Real.1 - (Real.1 - t2) * t3) = Real.0
        (Real.1 - t1) * x * (Real.1 - (Real.1 - t2) * t3) +
            t1 * y * (Real.1 - (Real.1 - t2) * t3) = Real.0
        // substitute the two elimination identities
        (Real.1 - t1) * ((Real.1 - t2) * (t3 - Real.1) * w) +
            t1 * (t2 * t3 * w) = Real.0
        // expand the products
        (Real.1 - t1) * ((Real.1 - t2) * (t3 - Real.1) * w) =
            (Real.1 - t1) * (Real.1 - t2) * (t3 - Real.1) * w
        (Real.1 - t1) * (Real.1 - t2) * (t3 - Real.1) * w + t1 * (t2 * t3 * w) = Real.0
        // t3 - 1 = -(1 - t3)
        t3 - Real.1 = -(Real.1 - t3)
        (Real.1 - t1) * (Real.1 - t2) * (t3 - Real.1) * w =
            (Real.1 - t1) * (Real.1 - t2) * (-(Real.1 - t3)) * w
        (Real.1 - t1) * (Real.1 - t2) * (-(Real.1 - t3)) * w =
            -((Real.1 - t1) * (Real.1 - t2) * (Real.1 - t3)) * w
        -((Real.1 - t1) * (Real.1 - t2) * (Real.1 - t3)) * w + t1 * (t2 * t3 * w) = Real.0
        t1 * (t2 * t3 * w) = t1 * t2 * t3 * w
        -((Real.1 - t1) * (Real.1 - t2) * (Real.1 - t3)) * w +
            t1 * t2 * t3 * w = Real.0
        // factor w out of the two terms
        -((Real.1 - t1) * (Real.1 - t2) * (Real.1 - t3)) * w + t1 * t2 * t3 * w =
            t1 * t2 * t3 * w + -((Real.1 - t1) * (Real.1 - t2) * (Real.1 - t3)) * w
        t1 * t2 * t3 * w + -((Real.1 - t1) * (Real.1 - t2) * (Real.1 - t3)) * w =
            (t1 * t2 * t3 + -((Real.1 - t1) * (Real.1 - t2) * (Real.1 - t3))) * w
        (t1 * t2 * t3 + -((Real.1 - t1) * (Real.1 - t2) * (Real.1 - t3))) * w =
            (t1 * t2 * t3 - (Real.1 - t1) * (Real.1 - t2) * (Real.1 - t3)) * w
        (t1 * t2 * t3 - (Real.1 - t1) * (Real.1 - t2) * (Real.1 - t3)) * w = Real.0
        w * (t1 * t2 * t3 - (Real.1 - t1) * (Real.1 - t2) * (Real.1 - t3)) = Real.0
    }
}

// ---------------------------------------------------------------------------
// Ceva's theorem.
// ---------------------------------------------------------------------------

/// Ceva's theorem: if the cevians `ad`, `be`, `cf` of the triangle `abc` are
/// concurrent at `p`, then the product of the directed ratios in which they
/// divide the sides equals one: `t1 t2 t3 = (1 - t1)(1 - t2)(1 - t3)` with
/// `d = (1 - t1) b + t1 c`, `e = (1 - t2) c + t2 a` and
/// `f = (1 - t3) a + t3 b`, i.e. `(BD/DC)(CE/EA)(AF/FB) = 1`.
theorem theorems1000_ceva(
    a: Pair[Real, Real], b: Pair[Real, Real], c: Pair[Real, Real],
    d: Pair[Real, Real], e: Pair[Real, Real], f: Pair[Real, Real],
    p: Pair[Real, Real], t1: Real, t2: Real, t3: Real
) {
    r2_add(r2_smul(Real.1 - t1, b), r2_smul(t1, c)) = d and
    r2_add(r2_smul(Real.1 - t2, c), r2_smul(t2, a)) = e and
    r2_add(r2_smul(Real.1 - t3, a), r2_smul(t3, b)) = f and
    r2_cross(r2_sub(p, a), r2_sub(d, a)) = Real.0 and
    r2_cross(r2_sub(p, b), r2_sub(e, b)) = Real.0 and
    r2_cross(r2_sub(p, c), r2_sub(f, c)) = Real.0 and
    r2_cross(r2_sub(b, a), r2_sub(c, a)) != Real.0
    implies
    t1 * t2 * t3 = (Real.1 - t1) * (Real.1 - t2) * (Real.1 - t3)
} by {
    if r2_add(r2_smul(Real.1 - t1, b), r2_smul(t1, c)) = d and
        r2_add(r2_smul(Real.1 - t2, c), r2_smul(t2, a)) = e and
        r2_add(r2_smul(Real.1 - t3, a), r2_smul(t3, b)) = f and
        r2_cross(r2_sub(p, a), r2_sub(d, a)) = Real.0 and
        r2_cross(r2_sub(p, b), r2_sub(e, b)) = Real.0 and
        r2_cross(r2_sub(p, c), r2_sub(f, c)) = Real.0 and
        r2_cross(r2_sub(b, a), r2_sub(c, a)) != Real.0 {
        r2_add(r2_smul(Real.1 - t1, b), r2_smul(t1, c)) = d
        r2_add(r2_smul(Real.1 - t2, c), r2_smul(t2, a)) = e
        r2_add(r2_smul(Real.1 - t3, a), r2_smul(t3, b)) = f
        r2_cross(r2_sub(p, a), r2_sub(d, a)) = Real.0
        r2_cross(r2_sub(p, b), r2_sub(e, b)) = Real.0
        r2_cross(r2_sub(p, c), r2_sub(f, c)) = Real.0
        r2_cross(r2_sub(b, a), r2_sub(c, a)) != Real.0
        // d - a = (1 - t1)(b - a) + t1 (c - a)
        ceva_d_sub_a(a, b, c, d, t1)
        r2_sub(d, a) =
            r2_add(
                r2_smul(Real.1 - t1, r2_sub(b, a)),
                r2_smul(t1, r2_sub(c, a)))
        // cross(p - a, d - a) = 0 expands to (1 - t1) X + t1 Y = 0
        r2_cross(r2_sub(p, a), r2_sub(d, a)) =
            r2_cross(
                r2_sub(p, a),
                r2_add(
                    r2_smul(Real.1 - t1, r2_sub(b, a)),
                    r2_smul(t1, r2_sub(c, a))))
        ceva_cross_pd(p, a, b, c, t1)
        r2_cross(
            r2_sub(p, a),
            r2_add(
                r2_smul(Real.1 - t1, r2_sub(b, a)),
                r2_smul(t1, r2_sub(c, a)))) =
            (Real.1 - t1) * r2_cross(r2_sub(p, a), r2_sub(b, a)) +
                t1 * r2_cross(r2_sub(p, a), r2_sub(c, a))
        (Real.1 - t1) * r2_cross(r2_sub(p, a), r2_sub(b, a)) +
            t1 * r2_cross(r2_sub(p, a), r2_sub(c, a)) = Real.0
        // p - b = (p - a) - (b - a),  e - b = (1 - t2)(c - a) - (b - a)
        r2_sub_sub_same_base(p, a, b)
        r2_sub(r2_sub(p, a), r2_sub(b, a)) = r2_sub(p, b)
        ceva_e_sub_b(a, b, c, e, t2)
        r2_sub(e, b) =
            r2_sub(r2_smul(Real.1 - t2, r2_sub(c, a)), r2_sub(b, a))
        // cross(p - b, e - b) = 0 expands to (1 - t2) Y - X - (1 - t2) W = 0
        r2_cross(r2_sub(p, b), r2_sub(e, b)) =
            r2_cross(
                r2_sub(r2_sub(p, a), r2_sub(b, a)),
                r2_sub(r2_smul(Real.1 - t2, r2_sub(c, a)), r2_sub(b, a)))
        ceva_cross_pb_eb(p, a, b, c, t2)
        r2_cross(
            r2_sub(r2_sub(p, a), r2_sub(b, a)),
            r2_sub(r2_smul(Real.1 - t2, r2_sub(c, a)), r2_sub(b, a))) =
            (Real.1 - t2) * r2_cross(r2_sub(p, a), r2_sub(c, a)) -
                r2_cross(r2_sub(p, a), r2_sub(b, a)) -
                (Real.1 - t2) * r2_cross(r2_sub(b, a), r2_sub(c, a))
        (Real.1 - t2) * r2_cross(r2_sub(p, a), r2_sub(c, a)) -
            r2_cross(r2_sub(p, a), r2_sub(b, a)) -
            (Real.1 - t2) * r2_cross(r2_sub(b, a), r2_sub(c, a)) = Real.0
        ceva_lin1(
            r2_cross(r2_sub(p, a), r2_sub(b, a)),
            r2_cross(r2_sub(p, a), r2_sub(c, a)),
            r2_cross(r2_sub(b, a), r2_sub(c, a)),
            t2)
        r2_cross(r2_sub(p, a), r2_sub(b, a)) =
            (Real.1 - t2) *
                (r2_cross(r2_sub(p, a), r2_sub(c, a)) -
                    r2_cross(r2_sub(b, a), r2_sub(c, a)))
        // p - c = (p - a) - (c - a),  f - c = t3 (b - a) - (c - a)
        r2_sub_sub_same_base(p, a, c)
        r2_sub(r2_sub(p, a), r2_sub(c, a)) = r2_sub(p, c)
        ceva_f_sub_c(a, b, c, f, t3)
        r2_sub(f, c) =
            r2_sub(r2_smul(t3, r2_sub(b, a)), r2_sub(c, a))
        // cross(p - c, f - c) = 0 expands to t3 X - Y + t3 W = 0
        r2_cross(r2_sub(p, c), r2_sub(f, c)) =
            r2_cross(
                r2_sub(r2_sub(p, a), r2_sub(c, a)),
                r2_sub(r2_smul(t3, r2_sub(b, a)), r2_sub(c, a)))
        ceva_cross_pc_fc(p, a, b, c, t3)
        r2_cross(
            r2_sub(r2_sub(p, a), r2_sub(c, a)),
            r2_sub(r2_smul(t3, r2_sub(b, a)), r2_sub(c, a))) =
            t3 * r2_cross(r2_sub(p, a), r2_sub(b, a)) -
                r2_cross(r2_sub(p, a), r2_sub(c, a)) +
                t3 * r2_cross(r2_sub(b, a), r2_sub(c, a))
        t3 * r2_cross(r2_sub(p, a), r2_sub(b, a)) -
            r2_cross(r2_sub(p, a), r2_sub(c, a)) +
            t3 * r2_cross(r2_sub(b, a), r2_sub(c, a)) = Real.0
        ceva_lin2(
            r2_cross(r2_sub(p, a), r2_sub(b, a)),
            r2_cross(r2_sub(p, a), r2_sub(c, a)),
            r2_cross(r2_sub(b, a), r2_sub(c, a)),
            t3)
        r2_cross(r2_sub(p, a), r2_sub(c, a)) =
            t3 *
                (r2_cross(r2_sub(p, a), r2_sub(b, a)) +
                    r2_cross(r2_sub(b, a), r2_sub(c, a)))
        // the elimination identities
        ceva_elim_x(
            r2_cross(r2_sub(p, a), r2_sub(b, a)),
            r2_cross(r2_sub(p, a), r2_sub(c, a)),
            r2_cross(r2_sub(b, a), r2_sub(c, a)),
            t2, t3)
        r2_cross(r2_sub(p, a), r2_sub(b, a)) *
            (Real.1 - (Real.1 - t2) * t3) =
            (Real.1 - t2) * (t3 - Real.1) * r2_cross(r2_sub(b, a), r2_sub(c, a))
        ceva_elim_y(
            r2_cross(r2_sub(p, a), r2_sub(b, a)),
            r2_cross(r2_sub(p, a), r2_sub(c, a)),
            r2_cross(r2_sub(b, a), r2_sub(c, a)),
            t2, t3)
        r2_cross(r2_sub(p, a), r2_sub(c, a)) *
            (Real.1 - (Real.1 - t2) * t3) =
            t2 * t3 * r2_cross(r2_sub(b, a), r2_sub(c, a))
        ceva_product_zero(
            t1, t2, t3,
            r2_cross(r2_sub(p, a), r2_sub(b, a)),
            r2_cross(r2_sub(p, a), r2_sub(c, a)),
            r2_cross(r2_sub(b, a), r2_sub(c, a)))
        r2_cross(r2_sub(b, a), r2_sub(c, a)) *
            (t1 * t2 * t3 - (Real.1 - t1) * (Real.1 - t2) * (Real.1 - t3)) = Real.0
        // cancel the nonzero cross
        Real.0 =
            r2_cross(r2_sub(b, a), r2_sub(c, a)) *
                (t1 * t2 * t3 - (Real.1 - t1) * (Real.1 - t2) * (Real.1 - t3))
        mul_left_cancel(
            Real.0,
            r2_cross(r2_sub(b, a), r2_sub(c, a)),
            t1 * t2 * t3 - (Real.1 - t1) * (Real.1 - t2) * (Real.1 - t3))
        Real.0 / r2_cross(r2_sub(b, a), r2_sub(c, a)) =
            t1 * t2 * t3 - (Real.1 - t1) * (Real.1 - t2) * (Real.1 - t3)
        Real.0 / r2_cross(r2_sub(b, a), r2_sub(c, a)) = Real.0
        t1 * t2 * t3 - (Real.1 - t1) * (Real.1 - t2) * (Real.1 - t3) = Real.0
        t1 * t2 * t3 = (Real.1 - t1) * (Real.1 - t2) * (Real.1 - t3)
    }
}
