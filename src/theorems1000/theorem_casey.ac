from pair import Pair
from real import Real
from top100 import r2_sub

// Casey's theorem: a generalization of Ptolemy's theorem.  Given four
// circles each externally tangent to two of the others in a ring, if `t_ij`
// denotes the length of the common external tangent segment of the circles
// `i` and `j`, then
//     t12 * t34 + t23 * t41 = t13 * t24.
// Ptolemy's theorem is the degenerate case in which the four circles are
// points (zero radius): the tangent lengths become the pairwise distances
// and the identity is exactly Ptolemy's relation for a cyclic quadrilateral
// (recorded in `theorems1000_ptolemy` in this directory).
//
// The statement below records the four circles by their tangent-length
// witnesses `t12`, `t23`, `t34`, `t41`, `t13`, `t24` (six pairwise tangent
// lengths), with the tangency arrangement left implicit: the tangent length
// of two circles with centers `o_i`, `o_j` and radii `r_i`, `r_j` satisfies
// `t_ij^2 = |o_i - o_j|^2 - (r_i - r_j)^2` for external tangency and
// `t_ij^2 = |o_i - o_j|^2 - (r_i + r_j)^2` for internal tangency.  The
// conclusion is the real identity `t12 * t34 + t23 * t41 = t13 * t24`.
//
// Proof sketch: Casey proved the theorem by inversion, sending one of the
// four circles to a line, or by a limiting argument from Ptolemy's theorem:
// fix three of the four circles and let the fourth shrink to a point, then
// each tangent length tends to the corresponding distance from the point to
// a circle, and a further limit degenerating the three circles to points
// recovers Ptolemy.  Alternatively, applying an inversion centered at a
// point of tangency of two of the circles turns the two circles into
// parallel lines and the other two into circles, reducing the identity to
// the tangent-length formula for two circles and two lines.  In
// coordinates, with the four centers and radii given, both sides expand to
// the same combination of the squared center distances, but the expansion
// of the six square roots (the tangent lengths themselves, not their
// squares) is the obstruction: the identity is between the lengths, so a
// formal proof needs the square root function on the reals and the
// elimination of the radicals by squaring, which is not yet in the
// library.  The statement is therefore recorded with its witness
// formulation and the proof left commented out, exactly as for
// `theorems1000_ptolemy`.
//
// theorem theorems1000_casey(
//     o1: Pair[Real, Real], o2: Pair[Real, Real],
//     o3: Pair[Real, Real], o4: Pair[Real, Real],
//     t12: Real, t23: Real, t34: Real, t41: Real, t13: Real, t24: Real
// ) {
//     t12 * t12 = r2_norm_sq(r2_sub(o2, o1)) and
//     t23 * t23 = r2_norm_sq(r2_sub(o3, o2)) and
//     t34 * t34 = r2_norm_sq(r2_sub(o4, o3)) and
//     t41 * t41 = r2_norm_sq(r2_sub(o1, o4)) and
//     t13 * t13 = r2_norm_sq(r2_sub(o3, o1)) and
//     t24 * t24 = r2_norm_sq(r2_sub(o4, o2))
//     implies
//     t12 * t34 + t23 * t41 = t13 * t24
// }
