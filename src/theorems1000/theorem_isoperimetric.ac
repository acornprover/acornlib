// The isoperimetric theorem (the classical form is due to the ancient
// Greeks — Zenodorus, c. 200 BC, proved the polygonal case; the modern
// theorem was proved by Jakob Steiner, 1838, and given a complete
// existence proof by Karl Weierstrass, 1870): among all closed plane
// curves with a given perimeter, the circle encloses the largest area.
// Equivalently, for a simple closed curve of length `L` enclosing area
// `A`,
//     4 π A ≤ L²,
// with equality exactly for the circle.  The inequality is the model
// example of a geometric variational problem: the optimal shape is found
// by symmetrization (Steiner's argument shows that any non-circular
// maximizer can be improved, the hard part being the existence of a
// maximizer, supplied by the compactness theorems of Weierstrass).  The
// theorem has generalizations to higher dimensions (the ball maximizes
// volume among bodies of given surface area), to Riemannian manifolds,
// and to the discrete setting of graphs (the edge-isoperimetric
// inequality).
//
// A formal statement needs a theory of plane curves with length and
// enclosed area — e.g. via the length of a parametrized curve and
// Green's theorem (the area enclosed by a positively oriented simple
// closed curve `c` is `(1/2) ∮ (x dy - y dx)`), or via the Brunn–Minkowski
// inequality.  The library has no curve or area machinery beyond the
// analytic calculus that is private to the `real` package, so the
// statement is recorded as prose with a schematic skeleton.  The entry is
// the "Isoperimetric theorem" of the "1000+ theorems" list.
//
// Proof sketch (the classical two-step argument): (1) existence — among
// curves of length `L`, the enclosed area is bounded above (the curve lies
// in a disk of radius `L/2`), and a maximizing curve exists by a compactness
// argument; (2) the maximizer is a circle — if a maximizing curve is not a
// circle, Steiner symmetrization or the "flattening" argument produces a
// curve of the same length with larger area, a contradiction.  The
// analytic proof of the inequality proceeds by parametrizing the curve by
// arc length and applying Wirtinger's inequality to the coordinates.

// theorem theorems1000_isoperimetric {
//     // Schematic: for a simple closed plane curve with length L and
//     // enclosed area A,
//     //   from_nat[Real](Nat.4) * pi * A <= L * L
//     // with equality exactly when the curve is a circle.
// }
