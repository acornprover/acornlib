from real import Real
from theorems1000.r2_helpers import real_add_pair_rearrange

numerals Real

// Sophie Germain's identity: for all real numbers `a` and `b`,
//     a^4 + 4 b^4 = (a^2 + 2 a b + 2 b^2) (a^2 - 2 a b + 2 b^2).
// The identity is the algebraic core of Sophie Germain's theorem on
// Fermat's last theorem (see theorem_sophie_germain_prime.ac in this
// directory): when `a` and `b` are positive integers with `a > 1` or
// `b > 1`, both factors are integers greater than one, so `a^4 + 4 b^4`
// is composite.  Over the naturals the second factor is written as
// `(a^2 + 2 b^2) - 2 a b`, which is exact because
// `a^2 + 2 b^2 - 2 a b = (a - b)^2 + b^2 >= 0`.
//
// The proof below factors the left-hand side as a difference of squares:
//     (a^2 + 2 b^2 + 2 a b) (a^2 + 2 b^2 - 2 a b)
//         = (a^2 + 2 b^2)^2 - (2 a b)^2
//         = (a^4 + 4 a^2 b^2 + 4 b^4) - 4 a^2 b^2 = a^4 + 4 b^4.
// Each step is packaged as a lemma so that the final assembly is a short
// chain of citations.

/// The real number two, `1 + 1` (the digit `Real.2` is not defined in the
/// library).
let real_two: Real = Real.1 + Real.1

/// The real number four, `1 + 1 + 1 + 1` (the digit `Real.4` is not defined
/// in the library).
let real_four: Real = Real.1 + Real.1 + Real.1 + Real.1

/// Two times two is four.
theorem sg_two_times_two {
    real_two * real_two = real_four
} by {
    real_two * real_two = (Real.1 + Real.1) * (Real.1 + Real.1)
    (Real.1 + Real.1) * (Real.1 + Real.1) = Real.1 + Real.1 + Real.1 + Real.1
    Real.1 + Real.1 + Real.1 + Real.1 = real_four
    real_two * real_two = real_four
}

/// The product of conjugate linear factors is a difference of squares.
theorem sg_difference_of_squares(x: Real, y: Real) {
    (x + y) * (x - y) = x * x - y * y
} by {
    (x + y) * (x - y) = (x + y) * x - (x + y) * y
    (x + y) * x = x * x + y * x
    (x + y) * y = x * y + y * y
    (x + y) * (x - y) = x * x + y * x - (x * y + y * y)
    y * x = x * y
    x * x + y * x - (x * y + y * y) = x * x + x * y - (x * y + y * y)
    x * x + x * y - (x * y + y * y) = x * x + x * y - x * y - y * y
    x * x + x * y - x * y - y * y = x * x + (x * y - x * y) - y * y
    x * y - x * y = Real.0
    x * x + (x * y - x * y) - y * y = x * x - y * y
}

/// The square of a two-term sum.
theorem sg_square_add(x: Real, y: Real) {
    (x + y) * (x + y) = x * x + y * x + (x * y + y * y)
} by {
    (x + y) * (x + y) = (x + y) * x + (x + y) * y
    (x + y) * x = x * x + y * x
    (x + y) * y = x * y + y * y
    (x + y) * (x + y) = x * x + y * x + (x * y + y * y)
}

/// Regrouping the first factor: `a^2 + 2ab + 2b^2 = (a^2 + 2b^2) + 2ab`.
theorem sg_regroup_first(a: Real, b: Real) {
    a * a + real_two * a * b + real_two * (b * b) =
        (a * a + real_two * (b * b)) + real_two * a * b
} by {
    a * a + real_two * a * b + real_two * (b * b) =
        a * a + (real_two * a * b + real_two * (b * b))
    real_two * a * b + real_two * (b * b) = real_two * (b * b) + real_two * a * b
    a * a + (real_two * a * b + real_two * (b * b)) =
        (a * a + real_two * (b * b)) + real_two * a * b
}

/// Regrouping the second factor: `a^2 - 2ab + 2b^2 = (a^2 + 2b^2) - 2ab`.
theorem sg_regroup_second(a: Real, b: Real) {
    a * a - real_two * a * b + real_two * (b * b) =
        (a * a + real_two * (b * b)) - real_two * a * b
} by {
    a * a - real_two * a * b + real_two * (b * b) =
        (a * a - real_two * a * b) + real_two * (b * b)
    (a * a - real_two * a * b) + real_two * (b * b) =
        (a * a + real_two * (b * b)) - real_two * a * b
}

/// The square of `a^2 + 2b^2` expands to `a^4 + 2 a^2 b^2 + (2 a^2 b^2 + 4 b^4)`.
theorem sg_u_square(a: Real, b: Real) {
    (a * a + real_two * (b * b)) * (a * a + real_two * (b * b)) =
        a * a * (a * a) + real_two * (b * b) * (a * a) +
            (a * a * (real_two * (b * b)) + real_two * (b * b) * (real_two * (b * b)))
} by {
    sg_square_add(a * a, real_two * (b * b))
    (a * a + real_two * (b * b)) * (a * a + real_two * (b * b)) =
        a * a * (a * a) + real_two * (b * b) * (a * a) +
            (a * a * (real_two * (b * b)) + real_two * (b * b) * (real_two * (b * b)))
}

/// The two middle terms of the expansion both equal `2 a^2 b^2`.
theorem sg_middle_terms(a: Real, b: Real) {
    real_two * (b * b) * (a * a) = real_two * (a * a) * (b * b) and
    a * a * (real_two * (b * b)) = real_two * (a * a) * (b * b)
} by {
    real_two * (b * b) * (a * a) = real_two * (a * a) * (b * b)
    a * a * (real_two * (b * b)) = real_two * (a * a) * (b * b)
    real_two * (b * b) * (a * a) = real_two * (a * a) * (b * b) and
        a * a * (real_two * (b * b)) = real_two * (a * a) * (b * b)
}

/// Twice `2 a^2 b^2` is `4 a^2 b^2`.
theorem sg_middle_sum(a: Real, b: Real) {
    real_two * (a * a) * (b * b) + real_two * (a * a) * (b * b) =
        real_four * (a * a) * (b * b)
} by {
    real_two * (a * a) * (b * b) + real_two * (a * a) * (b * b) =
        (real_two + real_two) * (a * a * (b * b))
    real_two + real_two = real_four
    (real_two + real_two) * (a * a * (b * b)) = real_four * (a * a) * (b * b)
}

/// The last term of the expansion is `4 b^4`.
theorem sg_last_term(a: Real, b: Real) {
    real_two * (b * b) * (real_two * (b * b)) = real_four * (b * b * b * b)
} by {
    real_two * (b * b) * (real_two * (b * b)) = real_two * real_two * (b * b * (b * b))
    sg_two_times_two
    real_two * real_two = real_four
    real_two * real_two * (b * b * (b * b)) = real_four * (b * b * (b * b))
    b * b * (b * b) = b * b * b * b
    real_four * (b * b * (b * b)) = real_four * (b * b * b * b)
}

/// The square of `2ab` is `4 a^2 b^2`.
theorem sg_v_square(a: Real, b: Real) {
    (real_two * a * b) * (real_two * a * b) = real_four * (a * a) * (b * b)
} by {
    (real_two * a * b) * (real_two * a * b) = real_two * real_two * (a * b * (a * b))
    sg_two_times_two
    real_two * real_two = real_four
    real_two * real_two * (a * b * (a * b)) = real_four * (a * b * (a * b))
    a * b * (a * b) = a * a * (b * b)
    real_four * (a * b * (a * b)) = real_four * (a * a * (b * b))
    real_four * (a * a * (b * b)) = real_four * (a * a) * (b * b)
}

/// The three expansion terms of `(a^2 + 2b^2)^2` combine into
/// `4 a^2 b^2 + 4 b^4`.
theorem sg_expansion_combine(a: Real, b: Real) {
    real_two * (b * b) * (a * a) +
        (a * a * (real_two * (b * b)) + real_two * (b * b) * (real_two * (b * b))) =
        real_four * (a * a) * (b * b) + real_four * (b * b * b * b)
} by {
    sg_middle_terms(a, b)
    real_two * (b * b) * (a * a) = real_two * (a * a) * (b * b)
    a * a * (real_two * (b * b)) = real_two * (a * a) * (b * b)
    real_two * (b * b) * (a * a) +
        (a * a * (real_two * (b * b)) + real_two * (b * b) * (real_two * (b * b))) =
        real_two * (a * a) * (b * b) +
            (real_two * (a * a) * (b * b) + real_two * (b * b) * (real_two * (b * b)))
    real_two * (a * a) * (b * b) +
        (real_two * (a * a) * (b * b) + real_two * (b * b) * (real_two * (b * b))) =
        (real_two * (a * a) * (b * b) + real_two * (a * a) * (b * b)) +
            real_two * (b * b) * (real_two * (b * b))
    sg_middle_sum(a, b)
    real_two * (a * a) * (b * b) + real_two * (a * a) * (b * b) =
        real_four * (a * a) * (b * b)
    (real_two * (a * a) * (b * b) + real_two * (a * a) * (b * b)) +
        real_two * (b * b) * (real_two * (b * b)) =
        real_four * (a * a) * (b * b) + real_two * (b * b) * (real_two * (b * b))
    sg_last_term(a, b)
    real_two * (b * b) * (real_two * (b * b)) = real_four * (b * b * b * b)
    real_four * (a * a) * (b * b) + real_two * (b * b) * (real_two * (b * b)) =
        real_four * (a * a) * (b * b) + real_four * (b * b * b * b)
}

/// Cancelling the doubled middle term: `a^4 + 4a^2b^2 + 4b^4 - 4a^2b^2 = a^4 + 4b^4`.
theorem sg_cancel_middle(a: Real, b: Real) {
    a * a * (a * a) + real_four * (a * a) * (b * b) +
        real_four * (b * b * b * b) - real_four * (a * a) * (b * b) =
        a * a * a * a + real_four * (b * b * b * b)
} by {
    a * a * (a * a) + real_four * (a * a) * (b * b) +
        real_four * (b * b * b * b) - real_four * (a * a) * (b * b) =
        a * a * (a * a) + real_four * (a * a) * (b * b) +
            real_four * (b * b * b * b) + -(real_four * (a * a) * (b * b))
    a * a * (a * a) + real_four * (a * a) * (b * b) +
        real_four * (b * b * b * b) + -(real_four * (a * a) * (b * b)) =
        (a * a * (a * a) + real_four * (a * a) * (b * b)) +
            (real_four * (b * b * b * b) + -(real_four * (a * a) * (b * b)))
    real_add_pair_rearrange(a * a * (a * a), real_four * (a * a) * (b * b),
        real_four * (b * b * b * b), -(real_four * (a * a) * (b * b)))
    (a * a * (a * a) + real_four * (a * a) * (b * b)) +
        (real_four * (b * b * b * b) + -(real_four * (a * a) * (b * b))) =
        (a * a * (a * a) + real_four * (b * b * b * b)) +
            (real_four * (a * a) * (b * b) + -(real_four * (a * a) * (b * b)))
    real_four * (a * a) * (b * b) + -(real_four * (a * a) * (b * b)) = Real.0
    (a * a * (a * a) + real_four * (b * b * b * b)) +
        (real_four * (a * a) * (b * b) + -(real_four * (a * a) * (b * b))) =
        (a * a * (a * a) + real_four * (b * b * b * b)) + Real.0
    (a * a * (a * a) + real_four * (b * b * b * b)) + Real.0 =
        a * a * a * a + real_four * (b * b * b * b)
}

/// Sophie Germain's identity: `a^4 + 4 b^4` factors as a product of two
/// sums of two squares, `(a^2 + 2 a b + 2 b^2)(a^2 - 2 a b + 2 b^2)`.
theorem theorems1000_sophie_germain(a: Real, b: Real) {
    (a * a + real_two * a * b + real_two * (b * b)) *
        (a * a - real_two * a * b + real_two * (b * b)) =
        a * a * a * a + real_four * (b * b * b * b)
} by {
    sg_regroup_first(a, b)
    a * a + real_two * a * b + real_two * (b * b) =
        (a * a + real_two * (b * b)) + real_two * a * b
    sg_regroup_second(a, b)
    a * a - real_two * a * b + real_two * (b * b) =
        (a * a + real_two * (b * b)) - real_two * a * b
    sg_difference_of_squares(a * a + real_two * (b * b), real_two * a * b)
    ((a * a + real_two * (b * b)) + real_two * a * b) *
        ((a * a + real_two * (b * b)) - real_two * a * b) =
        (a * a + real_two * (b * b)) * (a * a + real_two * (b * b)) -
            (real_two * a * b) * (real_two * a * b)
    sg_u_square(a, b)
    (a * a + real_two * (b * b)) * (a * a + real_two * (b * b)) =
        a * a * (a * a) + real_two * (b * b) * (a * a) +
            (a * a * (real_two * (b * b)) + real_two * (b * b) * (real_two * (b * b)))
    sg_v_square(a, b)
    (real_two * a * b) * (real_two * a * b) = real_four * (a * a) * (b * b)
    sg_expansion_combine(a, b)
    real_two * (b * b) * (a * a) +
        (a * a * (real_two * (b * b)) + real_two * (b * b) * (real_two * (b * b))) =
        real_four * (a * a) * (b * b) + real_four * (b * b * b * b)
    sg_cancel_middle(a, b)
    a * a * (a * a) + real_four * (a * a) * (b * b) +
        real_four * (b * b * b * b) - real_four * (a * a) * (b * b) =
        a * a * a * a + real_four * (b * b * b * b)
}
