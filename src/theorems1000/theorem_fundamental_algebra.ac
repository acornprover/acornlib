from complex import Complex, fundamental_theorem_of_algebra_of_extreme
from polynomial import Polynomial, polynomial_eval

// The fundamental theorem of algebra: every non-constant polynomial with
// complex coefficients has at least one complex root.  The first rigorous
// proofs are due to d'Alembert (1746) and Gauss (1799); the name "fundamental
// theorem of algebra" was coined by Gauss.  Over the real numbers the
// statement is false (`x^2 + 1` has no real root), which is why the
// complex numbers are the natural setting.
//
// The statement is recorded with the library's univariate polynomials
// `Polynomial[Complex]` (finitely supported coefficient functions) and
// evaluation `polynomial_eval(p, z)`.  A polynomial is non-constant when
// it takes two distinct values, written `exists(x, y) {
// polynomial_eval(p, x) != polynomial_eval(p, y) }`; a root is a value
// `z` with `polynomial_eval(p, z) = Complex.0`.
//
// Proof (minimum-modulus route, developed in `complex/fta_general.ac`):
// the perturbation lemma (D'Alembert's lemma) is proved in full: if
// `p(z0) != 0` and `p` is nonconstant, some nearby point evaluates to
// strictly smaller modulus.  The growth of `|p(z)|` at infinity and the
// iterated-quotient expansion around any point are also proved.  The
// remaining analytic ingredient is the extreme value theorem: `|p|`
// attains a minimum on a closed disk (equivalently, a global minimum,
// using the growth of `|p|`).  The library does not yet provide
// compactness of closed disks in the complex plane (no `TopologicalSpace`
// instance for `Complex`, no two-dimensional Heine-Borel, and no
// Bolzano-Weierstrass theorem), so the theorem below carries the extreme
// value property as an explicit hypothesis.  Once the extreme value
// theorem for `|p|` on a closed disk is available, the hypothesis
// `extreme_value_hypothesis` below is discharged by
// `global_minimum_from_disk` in `complex/fta_general.ac`.
//
// The theorem proved here (`fundamental_theorem_of_algebra_of_extreme`)
// shows that a global minimum of `|p|` is a root of `p`.

/// The extreme value hypothesis: `|p|` attains a global minimum.
define extreme_value_hypothesis(p: Polynomial[Complex]) -> Bool {
    exists(z0: Complex) {
        forall(z: Complex) { polynomial_eval(p, z0).modulus <= polynomial_eval(p, z).modulus }
    }
}

/// The fundamental theorem of algebra, modulo the extreme value theorem.
theorem theorems1000_fundamental_algebra_of_extreme(p: Polynomial[Complex]) {
    extreme_value_hypothesis(p) and
    exists(x: Complex, y: Complex) {
        polynomial_eval(p, x) != polynomial_eval(p, y)
    } implies exists(z: Complex) {
        polynomial_eval(p, z) = Complex.0
    }
} by {
    if extreme_value_hypothesis(p) and
        exists(x: Complex, y: Complex) {
            polynomial_eval(p, x) != polynomial_eval(p, y)
        } {
        extreme_value_hypothesis(p) = exists(z0: Complex) {
            forall(z: Complex) { polynomial_eval(p, z0).modulus <= polynomial_eval(p, z).modulus }
        }
        exists(z0: Complex) {
            forall(z: Complex) { polynomial_eval(p, z0).modulus <= polynomial_eval(p, z).modulus }
        }
        fundamental_theorem_of_algebra_of_extreme(p)
        exists(x: Complex, y: Complex) { polynomial_eval(p, x) != polynomial_eval(p, y) } and
            exists(z0: Complex) {
                forall(z: Complex) { polynomial_eval(p, z0).modulus <= polynomial_eval(p, z).modulus }
            }
        exists(z: Complex) { polynomial_eval(p, z) = Complex.0 }
    }
}

// The full statement, recorded without proof: it needs the extreme value
// theorem on closed disks of the complex plane (two-dimensional
// compactness), which the library does not yet provide.
//
// theorem theorems1000_fundamental_algebra(p: Polynomial[Complex]) {
//     exists(x: Complex, y: Complex) {
//         polynomial_eval(p, x) != polynomial_eval(p, y)
//     } implies exists(z: Complex) {
//         polynomial_eval(p, z) = Complex.0
//     }
// }
