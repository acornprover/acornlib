from pair import Pair
from real import Real, mul_left_cancel
from top100 import r2_add, r2_sub, r2_norm_sq
from theorems1000.r2_helpers import r2_smul, r2_cross, r2_add_sub_left_cancel, r2_sub_sub_same_base,
    r2_cross_sub_left, r2_cross_sub_right, r2_cross_smul_left, r2_cross_self_zero,
    r2_cross_swap, r2_norm_sq_smul

// The intercept theorem (Thales' intercept theorem, Strahlensatz): if two
// rays from a common vertex `s` are cut by two parallel lines, then the
// segments cut on the two rays are proportional.  Here `a` and `a'` lie on
// the first ray through `s` and `a'` (so `a = s + alpha (a' - s)` for the
// scalar `alpha`), `b` and `b'` lie on the second ray through `s` and `b'`
// (so `b = s + beta (b' - s)`), and the two cutting lines are parallel:
// `cross(b - a, b' - a') = 0`.  The theorem states the proportionality of
// the squared segment lengths:
//     |a - s|^2 * |b' - s|^2 = |a' - s|^2 * |b - s|^2.
//
// Proof sketch: with `u = a' - s` and `v = b' - s`, the hypotheses give
// `a - s = alpha u` and `b - s = beta v`, so `b - a = beta v - alpha u` and
// `b' - a' = v - u`.  Expanding the parallelity condition with the cross
// product bilinearity lemmas gives
//     (beta - alpha) * cross(v, u) = 0,
// and since `cross(v, u) != 0` (the two rays are distinct), `alpha = beta`
// by cancellation.  Finally `|a - s|^2 = alpha^2 |u|^2` and
// `|b - s|^2 = alpha^2 |v|^2`, so both sides of the conclusion equal
// `alpha^2 |u|^2 |v|^2`.

/// The intercept theorem: parallel cuts of two rays give proportional
/// segments.
///
/// With `a = s + alpha (a' - s)` on the ray through `a'`, `b = s + beta (b' - s)`
/// on the ray through `b'`, and the parallel lines `ab` and `a'b'`
/// (`cross(b - a, b' - a') = 0`), the squared segment lengths are
/// proportional: |a - s|^2 * |b' - s|^2 = |a' - s|^2 * |b - s|^2.
/// The two rays are assumed distinct: `cross(a' - s, b' - s) != 0`.
theorem theorems1000_intercept(
    s: Pair[Real, Real],
    a: Pair[Real, Real], a1: Pair[Real, Real],
    b: Pair[Real, Real], b1: Pair[Real, Real],
    alpha: Real, beta: Real
) {
    r2_add(s, r2_smul(alpha, r2_sub(a1, s))) = a and
    r2_add(s, r2_smul(beta, r2_sub(b1, s))) = b and
    r2_cross(r2_sub(b, a), r2_sub(b1, a1)) = Real.0 and
    r2_cross(r2_sub(a1, s), r2_sub(b1, s)) != Real.0
    implies
    r2_norm_sq(r2_sub(a, s)) * r2_norm_sq(r2_sub(b1, s)) =
        r2_norm_sq(r2_sub(a1, s)) * r2_norm_sq(r2_sub(b, s))
} by {
    if r2_add(s, r2_smul(alpha, r2_sub(a1, s))) = a and
        r2_add(s, r2_smul(beta, r2_sub(b1, s))) = b and
        r2_cross(r2_sub(b, a), r2_sub(b1, a1)) = Real.0 and
        r2_cross(r2_sub(a1, s), r2_sub(b1, s)) != Real.0 {
        r2_add(s, r2_smul(alpha, r2_sub(a1, s))) = a
        r2_add(s, r2_smul(beta, r2_sub(b1, s))) = b
        r2_cross(r2_sub(b, a), r2_sub(b1, a1)) = Real.0
        r2_cross(r2_sub(a1, s), r2_sub(b1, s)) != Real.0
        // a - s = alpha (a' - s) and b - s = beta (b' - s)
        r2_add_sub_left_cancel(s, r2_smul(alpha, r2_sub(a1, s)))
        r2_sub(r2_add(s, r2_smul(alpha, r2_sub(a1, s))), s) = r2_smul(alpha, r2_sub(a1, s))
        r2_sub(a, s) = r2_smul(alpha, r2_sub(a1, s))
        r2_add_sub_left_cancel(s, r2_smul(beta, r2_sub(b1, s)))
        r2_sub(r2_add(s, r2_smul(beta, r2_sub(b1, s))), s) = r2_smul(beta, r2_sub(b1, s))
        r2_sub(b, s) = r2_smul(beta, r2_sub(b1, s))
        // b - a = beta (b' - s) - alpha (a' - s)
        r2_sub_sub_same_base(s, a, b)
        r2_sub(r2_sub(b, s), r2_sub(a, s)) = r2_sub(b, a)
        r2_sub(b, a) = r2_sub(r2_smul(beta, r2_sub(b1, s)), r2_smul(alpha, r2_sub(a1, s)))
        // b' - a' = (b' - s) - (a' - s)
        r2_sub_sub_same_base(s, a1, b1)
        r2_sub(r2_sub(b1, s), r2_sub(a1, s)) = r2_sub(b1, a1)
        r2_sub(b1, a1) = r2_sub(r2_sub(b1, s), r2_sub(a1, s))
        // the parallelity condition in the u, v notation
        r2_cross(r2_sub(b, a), r2_sub(b1, a1)) =
            r2_cross(r2_sub(r2_smul(beta, r2_sub(b1, s)), r2_smul(alpha, r2_sub(a1, s))), r2_sub(r2_sub(b1, s), r2_sub(a1, s)))
        r2_cross(r2_sub(r2_smul(beta, r2_sub(b1, s)), r2_smul(alpha, r2_sub(a1, s))), r2_sub(r2_sub(b1, s), r2_sub(a1, s))) = Real.0
        // expand: cross(beta v - alpha u, v - u) = beta cross(v, v-u) - alpha cross(u, v-u)
        r2_cross_sub_left(r2_smul(beta, r2_sub(b1, s)), r2_smul(alpha, r2_sub(a1, s)), r2_sub(r2_sub(b1, s), r2_sub(a1, s)))
        r2_cross(r2_sub(r2_smul(beta, r2_sub(b1, s)), r2_smul(alpha, r2_sub(a1, s))), r2_sub(r2_sub(b1, s), r2_sub(a1, s))) =
            r2_cross(r2_smul(beta, r2_sub(b1, s)), r2_sub(r2_sub(b1, s), r2_sub(a1, s))) -
            r2_cross(r2_smul(alpha, r2_sub(a1, s)), r2_sub(r2_sub(b1, s), r2_sub(a1, s)))
        r2_cross_smul_left(beta, r2_sub(b1, s), r2_sub(r2_sub(b1, s), r2_sub(a1, s)))
        r2_cross(r2_smul(beta, r2_sub(b1, s)), r2_sub(r2_sub(b1, s), r2_sub(a1, s))) =
            beta * r2_cross(r2_sub(b1, s), r2_sub(r2_sub(b1, s), r2_sub(a1, s)))
        r2_cross_smul_left(alpha, r2_sub(a1, s), r2_sub(r2_sub(b1, s), r2_sub(a1, s)))
        r2_cross(r2_smul(alpha, r2_sub(a1, s)), r2_sub(r2_sub(b1, s), r2_sub(a1, s))) =
            alpha * r2_cross(r2_sub(a1, s), r2_sub(r2_sub(b1, s), r2_sub(a1, s)))
        // cross(v, v - u) = -cross(v, u) and cross(u, v - u) = cross(u, v)
        r2_cross_sub_right(r2_sub(b1, s), r2_sub(b1, s), r2_sub(a1, s))
        r2_cross(r2_sub(b1, s), r2_sub(r2_sub(b1, s), r2_sub(a1, s))) =
            r2_cross(r2_sub(b1, s), r2_sub(b1, s)) - r2_cross(r2_sub(b1, s), r2_sub(a1, s))
        r2_cross_self_zero(r2_sub(b1, s))
        r2_cross(r2_sub(b1, s), r2_sub(b1, s)) = Real.0
        r2_cross(r2_sub(b1, s), r2_sub(r2_sub(b1, s), r2_sub(a1, s))) =
            Real.0 - r2_cross(r2_sub(b1, s), r2_sub(a1, s))
        r2_cross_swap(r2_sub(a1, s), r2_sub(b1, s))
        r2_cross(r2_sub(a1, s), r2_sub(b1, s)) = -r2_cross(r2_sub(b1, s), r2_sub(a1, s))
        r2_cross_sub_right(r2_sub(a1, s), r2_sub(b1, s), r2_sub(a1, s))
        r2_cross(r2_sub(a1, s), r2_sub(r2_sub(b1, s), r2_sub(a1, s))) =
            r2_cross(r2_sub(a1, s), r2_sub(b1, s)) - r2_cross(r2_sub(a1, s), r2_sub(a1, s))
        r2_cross_self_zero(r2_sub(a1, s))
        r2_cross(r2_sub(a1, s), r2_sub(a1, s)) = Real.0
        r2_cross(r2_sub(a1, s), r2_sub(r2_sub(b1, s), r2_sub(a1, s))) =
            r2_cross(r2_sub(a1, s), r2_sub(b1, s)) - Real.0
        r2_cross(r2_sub(a1, s), r2_sub(b1, s)) - Real.0 = r2_cross(r2_sub(a1, s), r2_sub(b1, s))
        // both cross(v, v - u) and cross(u, v - u) equal cross(u, v)
        r2_cross(r2_sub(b1, s), r2_sub(r2_sub(b1, s), r2_sub(a1, s))) =
            r2_cross(r2_sub(a1, s), r2_sub(b1, s))
        r2_cross(r2_sub(a1, s), r2_sub(r2_sub(b1, s), r2_sub(a1, s))) =
            r2_cross(r2_sub(a1, s), r2_sub(b1, s))
        // scale both sides
        beta * r2_cross(r2_sub(b1, s), r2_sub(r2_sub(b1, s), r2_sub(a1, s))) =
            beta * r2_cross(r2_sub(a1, s), r2_sub(b1, s))
        alpha * r2_cross(r2_sub(a1, s), r2_sub(r2_sub(b1, s), r2_sub(a1, s))) =
            alpha * r2_cross(r2_sub(a1, s), r2_sub(b1, s))
        // assemble the whole cross product
        r2_cross(r2_sub(r2_smul(beta, r2_sub(b1, s)), r2_smul(alpha, r2_sub(a1, s))), r2_sub(r2_sub(b1, s), r2_sub(a1, s))) =
            r2_cross(r2_smul(beta, r2_sub(b1, s)), r2_sub(r2_sub(b1, s), r2_sub(a1, s))) -
            r2_cross(r2_smul(alpha, r2_sub(a1, s)), r2_sub(r2_sub(b1, s), r2_sub(a1, s)))
        r2_cross(r2_sub(r2_smul(beta, r2_sub(b1, s)), r2_smul(alpha, r2_sub(a1, s))), r2_sub(r2_sub(b1, s), r2_sub(a1, s))) =
            beta * r2_cross(r2_sub(b1, s), r2_sub(r2_sub(b1, s), r2_sub(a1, s))) -
            alpha * r2_cross(r2_sub(a1, s), r2_sub(r2_sub(b1, s), r2_sub(a1, s)))
        r2_cross(r2_sub(r2_smul(beta, r2_sub(b1, s)), r2_smul(alpha, r2_sub(a1, s))), r2_sub(r2_sub(b1, s), r2_sub(a1, s))) =
            beta * r2_cross(r2_sub(a1, s), r2_sub(b1, s)) -
            alpha * r2_cross(r2_sub(a1, s), r2_sub(r2_sub(b1, s), r2_sub(a1, s)))
        r2_cross(r2_sub(r2_smul(beta, r2_sub(b1, s)), r2_smul(alpha, r2_sub(a1, s))), r2_sub(r2_sub(b1, s), r2_sub(a1, s))) =
            beta * r2_cross(r2_sub(a1, s), r2_sub(b1, s)) -
            alpha * r2_cross(r2_sub(a1, s), r2_sub(b1, s))
        // factor: beta * c - alpha * c = (beta - alpha) * c
        beta * r2_cross(r2_sub(a1, s), r2_sub(b1, s)) -
            alpha * r2_cross(r2_sub(a1, s), r2_sub(b1, s)) =
            (beta - alpha) * r2_cross(r2_sub(a1, s), r2_sub(b1, s))
        r2_cross(r2_sub(r2_smul(beta, r2_sub(b1, s)), r2_smul(alpha, r2_sub(a1, s))), r2_sub(r2_sub(b1, s), r2_sub(a1, s))) =
            (beta - alpha) * r2_cross(r2_sub(a1, s), r2_sub(b1, s))
        // so (beta - alpha) * cross(u, v) = 0; with cross(u, v) != 0, alpha = beta
        (beta - alpha) * r2_cross(r2_sub(a1, s), r2_sub(b1, s)) = Real.0
        Real.0 = r2_cross(r2_sub(a1, s), r2_sub(b1, s)) * (beta - alpha)
        mul_left_cancel(Real.0, r2_cross(r2_sub(a1, s), r2_sub(b1, s)), beta - alpha)
        Real.0 / r2_cross(r2_sub(a1, s), r2_sub(b1, s)) = beta - alpha
        Real.0 / r2_cross(r2_sub(a1, s), r2_sub(b1, s)) = Real.0
        beta - alpha = Real.0
        alpha = beta
        // |a - s|^2 = alpha^2 |u|^2 and |b - s|^2 = alpha^2 |v|^2
        r2_norm_sq_smul(alpha, r2_sub(a1, s))
        r2_norm_sq(r2_smul(alpha, r2_sub(a1, s))) = alpha * alpha * r2_norm_sq(r2_sub(a1, s))
        r2_norm_sq(r2_sub(a, s)) = alpha * alpha * r2_norm_sq(r2_sub(a1, s))
        r2_norm_sq_smul(beta, r2_sub(b1, s))
        r2_norm_sq(r2_smul(beta, r2_sub(b1, s))) = beta * beta * r2_norm_sq(r2_sub(b1, s))
        r2_norm_sq(r2_sub(b, s)) = beta * beta * r2_norm_sq(r2_sub(b1, s))
        r2_norm_sq(r2_sub(b, s)) = alpha * alpha * r2_norm_sq(r2_sub(b1, s))
        // the conclusion: both sides equal alpha^2 |u|^2 |v|^2
        r2_norm_sq(r2_sub(a, s)) * r2_norm_sq(r2_sub(b1, s)) =
            (alpha * alpha * r2_norm_sq(r2_sub(a1, s))) * r2_norm_sq(r2_sub(b1, s))
        r2_norm_sq(r2_sub(a1, s)) * r2_norm_sq(r2_sub(b, s)) =
            r2_norm_sq(r2_sub(a1, s)) * (alpha * alpha * r2_norm_sq(r2_sub(b1, s)))
        (alpha * alpha * r2_norm_sq(r2_sub(a1, s))) * r2_norm_sq(r2_sub(b1, s)) =
            r2_norm_sq(r2_sub(a1, s)) * (alpha * alpha * r2_norm_sq(r2_sub(b1, s)))
        r2_norm_sq(r2_sub(a, s)) * r2_norm_sq(r2_sub(b1, s)) =
            r2_norm_sq(r2_sub(a1, s)) * r2_norm_sq(r2_sub(b, s))
    }
}
