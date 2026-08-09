from nat import Nat
from list import List, map

numerals Nat

// The Cauchy–Davenport theorem (Augustin-Louis Cauchy, 1813; re-proved by
// Harold Davenport in 1935): for a prime `p` and nonempty subsets `A`, `B`
// of the cyclic group `Z_p`,
//     |A + B| >= min(p, |A| + |B| - 1),
// where `A + B = { a + b (mod p) : a in A, b in B }` is the sumset.  The
// theorem is the fundamental lower bound of additive combinatorics: it
// determines exactly how small a sumset can be in a prime-order group.  The
// bound is sharp — if `A` and `B` are arithmetic progressions with the same
// common difference, then `|A + B| = |A| + |B| - 1` (when this does not
// exceed `p`).  It is the `p`-cyclic analogue of the trivial integer fact
// `|A + B| >= |A| + |B| - 1`, and it is the seed of the much later
// Polynomial method results (Dias da Silva–Hamidoune, the Erdős–Heilbronn
// conjecture).
//
// The formal statement below encodes `A` and `B` as unique lists of
// residues below `p`; the sumset `A + B` is the list of the residues
// `(a + b) mod p` over all pairs, and `|A + B|` is its deduplicated length
// (`.unique.length`).  `min` is the pointwise minimum on the naturals.
//
// Proof sketch (the classical induction of Cauchy): induct on `|A| + |B|`.
// If `|A| + |B| <= p + 1` there is nothing to show when the bound is
// `|A| + |B| - 1`; the interesting case is `|A| + |B| > p + 1`, where the
// claim is `|A + B| >= p`.  If some nonzero `d` satisfies
// `A + d ⊆ A` (so `A` is `d`-periodic), then `B` splits into residue
// classes modulo `d` and a counting argument over the classes gives
// `|A + B| >= p`.  Otherwise choose `d` with `a in A`, `a + d ∉ A`, and
// replace `A` by `A' = A ∪ (A + d)`: the sumset can only grow while
// `|A'| + |B|` decreased (the shift moves exactly one new element into
// `A`), so induction applies.  The library does not yet have a sumset
// operation over residues, so the statement is recorded without proof.

/// The pointwise minimum of two natural numbers.
define nat_min(a: Nat, b: Nat) -> Nat {
    if a <= b { a } else { b }
}

/// The summand `(a + b) mod p` for a fixed `a`, as a named function so that
/// `map` clients need not inline the lambda.
define summand_mod_fn(p: Nat, a: Nat) -> (Nat -> Nat) {
    function(b: Nat) { (a + b).mod(p) }
}

/// The row `{ (a + b) mod p : b in B }` of the sumset table for a fixed `a`.
define sumset_row(p: Nat, a: Nat, bs: List[Nat]) -> List[Nat] {
    map(bs, summand_mod_fn(p, a))
}

/// The sumset `A + B` of two residue lists modulo `p`: the list of
/// `(a + b) mod p` over all pairs, with multiplicities.  Its deduplicated
/// length is the cardinality `|A + B|` used in the theorem.
define sumset_mod(p: Nat, as: List[Nat], bs: List[Nat]) -> List[Nat] {
    match as {
        List.nil {
            List.nil[Nat]
        }
        List.cons(h, t) {
            sumset_mod(p, t, bs) + sumset_row(p, h, bs)
        }
    }
}

// theorem theorems1000_cauchy_davenport(p: Nat, A: List[Nat], B: List[Nat]) {
//     p.is_prime and A.length != Nat.0 and B.length != Nat.0 and
//     A.is_unique and B.is_unique and
//     (forall(x: Nat) { A.contains(x) implies x < p }) and
//     (forall(x: Nat) { B.contains(x) implies x < p })
//         implies nat_min(p, A.length + B.length - Nat.1) <=
//             sumset_mod(p, A, B).unique.length
// }
