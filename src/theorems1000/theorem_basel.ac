from nat import Nat, from_nat, from_nat_zero, from_nat_add, from_nat_one
from real import Real, converges, is_increasing, is_upper_bound, is_lower_bound, seq_lte,
    monotone_convergence_principle, partial_suc, partial_seq_lte,
    from_nat_suc_pos_real,
    mul_pos_pos, mul_le_mul_pos_left, mul_le_mul_pos_right,
    lte_add_right, neg_lte_flip, two_positive, lte_trans, real_mul_comm,
    mul_one_right, add_zero_right, real_no_zero_divisors, mul_div_cancel,
    lte_add_left, div_mul_cancel_right, from_nat_lte_mono
from list import partial, partial_split_last, partial_zero, partial_scalar_mul,
    partial_pointwise_eq
from order import lte_ref, lte_of_lt, lt_trans, lt_or_eq_of_lte
from ordered_field import inverse_on_positive_flips_inequality,
    inverse_of_positive_is_positive
from algebra.semigroup import mul_fn
from algebra.field.field import mul_not_zero

numerals Real

// The Basel problem: the sum of the reciprocals of the squares converges,
// and its value is pi^2 / 6:
//     sum_{n=1}^{infinity} 1 / n^2 = pi^2 / 6.
// The problem is named after Basel, the hometown of the Bernoulli family,
// who worked on it in the late seventeenth century; it resisted many
// attempts until Leonhard Euler solved it in 1734 (aged 28).  The value
// pi^2 / 6 also equals zeta(2), the value of the Riemann zeta function at 2,
// and it is the reciprocal of the probability that two random integers are
// coprime.
//
// The statement is recorded with the library's sequence notation: a real
// sequence `q: Nat -> Real` converges to `a` when `converges_to(q, a)`
// holds, and `partial(f, k)` is the sum `f(0) + ... + f(k - 1)`.  The
// partial sums of the squared reciprocals are indexed from `n = 0` with the
// term `1 / (n + 1)^2`, so the sum from `n = 1` to `k` is
// `partial(..., k)`, and the classical statement is
//     converges_to(partial sums of 1/(n+1)^2, pi^2/6).
//
// The exact value `pi^2 / 6` is not yet in the library, but the
// convergence itself is proved below as
// `theorems1000_basel_converges`: the partial sums form an increasing
// sequence bounded above by two.  The bound is the classical telescoping
// comparison `1 / (k + 1)^2 <= 2 (1 / (k + 1) - 1 / (k + 2))`: the
// majorant telescopes to `1 - 1 / (n + 1) <= 1`, so the partial sums are
// at most `2 * 1 = 2`.  The library proves the same convergence in
// `real/zeta_values.ac` (`zeta2_converges`, private to the `real`
// package); the proof below develops it from the public real analysis
// interface.

/// The kth term of the Basel series: 1 / (k + 1)^2, indexed from zero.
define basel_term(k: Nat) -> Real {
    Real.1 / (from_nat[Real](k.suc) * from_nat[Real](k.suc))
}

/// The kth telescoping majorant term: 1 / (k + 1) - 1 / (k + 2).
define basel_tel(k: Nat) -> Real {
    Real.1 / from_nat[Real](k.suc) - Real.1 / from_nat[Real](k.suc.suc)
}

/// The doubled telescoping majorant.
define basel_two_tel(k: Nat) -> Real {
    (Real.1 + Real.1) * basel_tel(k)
}

/// The reciprocal of a positive real is positive.
theorem basel_one_div_pos(a: Real) {
    a > Real.0 implies Real.1 / a > Real.0
} by {
    if a > Real.0 {
        a.is_positive
        inverse_of_positive_is_positive[Real](a)
        Real.0 < a.inverse
        a.inverse.is_positive
        Real.1.is_positive
        mul_pos_pos(Real.1, a.inverse)
        (Real.1 * a.inverse).is_positive
        Real.1 / a = Real.1 * a.inverse
        (Real.1 / a).is_positive
        Real.1 / a > Real.0
    }
}

/// Substituting equal numerators in a quotient: `x = y` gives
/// `x / f = y / f`.
theorem basel_div_subst(x: Real, y: Real, f: Real) {
    x = y implies x / f = y / f
} by {
    if x = y {
        x / f = y / f
    }
}

/// Substituting equal denominators in a quotient: `f = g` gives
/// `x / f = x / g`.
theorem basel_div_subst_den(x: Real, f: Real, g: Real) {
    f = g implies x / f = x / g
} by {
    if f = g {
        x / f = x / g
    }
}

/// Substituting equal sides in a non-strict inequality:
/// `x = u` and `y = v` and `u <= v` give `x <= y`.
theorem basel_lte_subst2(x: Real, y: Real, u: Real, v: Real) {
    x = u and y = v and u <= v implies x <= y
} by {
    if x = u and y = v and u <= v {
        x <= y
    }
}

/// Substituting equal subtrahends: `x = y` gives `a - x = a - y`.
theorem basel_sub_cong(a: Real, x: Real, y: Real) {
    x = y implies a - x = a - y
} by {
    if x = y {
        a - x = a - y
    }
}

/// Substituting in a sum: `x = u` and `y = v` give `x + y = u + v`.
theorem basel_add_subst2(x: Real, y: Real, u: Real, v: Real) {
    x = u and y = v implies x + y = u + v
} by {
    if x = u and y = v {
        x + y = u + v
    }
}

/// The partial sums of the doubled majorant double.
theorem basel_partial_two_tel(n: Nat) {
    partial(basel_two_tel, n) = (Real.1 + Real.1) * partial(basel_tel, n)
} by {
    partial_scalar_mul(Real.1 + Real.1, basel_tel, n)
    (Real.1 + Real.1) * partial(basel_tel, n) = partial(mul_fn(Real.1 + Real.1, basel_tel), n)
    forall(k: Nat) {
        if k < n {
            mul_fn(Real.1 + Real.1, basel_tel, k) = basel_two_tel(k)
        }
    }
    partial_pointwise_eq(mul_fn(Real.1 + Real.1, basel_tel), basel_two_tel, n)
    partial(mul_fn(Real.1 + Real.1, basel_tel), n) = partial(basel_two_tel, n)
    partial(basel_two_tel, n) = (Real.1 + Real.1) * partial(basel_tel, n)
}

/// Cancelling the middle term of the telescoping pair:
/// `(1 - a) + (a - b) = 1 - b`.
theorem basel_tel_cancel(a: Real, b: Real) {
    (Real.1 - a) + (a - b) = Real.1 - b
} by {
    (Real.1 - a) + (a - b) = Real.1 - a + a - b
    Real.1 - a + a - b = Real.1 - b
}

/// The negation of a positive real is at most zero.
theorem basel_neg_pos_le_zero(x: Real) {
    x > Real.0 implies -x <= Real.0
} by {
    if x > Real.0 {
        Real.0 <= x
        neg_lte_flip(Real.0, x)
        -x <= -Real.0
        -Real.0 = Real.0
        -x <= Real.0
    }
}

/// The real numbers zero and one are ordered: `0 <= 1`.
theorem basel_zero_le_one {
    Real.0 <= Real.1
} by {
    Real.0 < Real.1
    lte_of_lt[Real](Real.0, Real.1)
    Real.0 <= Real.1
}

/// The Basel terms are positive.
theorem basel_term_pos(k: Nat) {
    basel_term(k) > Real.0
} by {
    from_nat_suc_pos_real(k)
    from_nat[Real](k.suc) > Real.0
    mul_pos_pos(from_nat[Real](k.suc), from_nat[Real](k.suc))
    (from_nat[Real](k.suc) * from_nat[Real](k.suc)).is_positive
    basel_one_div_pos(from_nat[Real](k.suc) * from_nat[Real](k.suc))
    Real.1 / (from_nat[Real](k.suc) * from_nat[Real](k.suc)) > Real.0
    basel_term(k) > Real.0
}

/// The Basel terms are nonnegative.
theorem basel_term_nonneg(k: Nat) {
    Real.0 <= basel_term(k)
} by {
    basel_term_pos(k)
    basel_term(k) > Real.0
    Real.0 <= basel_term(k)
}

/// The partial sums of the telescoping majorant telescope:
/// `partial(basel_tel, n) = 1 - 1 / (n + 1)`.
theorem basel_tel_partial(n: Nat) {
    partial(basel_tel, n) = Real.1 - Real.1 / from_nat[Real](n.suc)
} by {
    define p(j: Nat) -> Bool {
        partial(basel_tel, j) = Real.1 - Real.1 / from_nat[Real](j.suc)
    }
    // base case: partial(tel, 0) = 0 = 1 - 1/1
    partial_zero(basel_tel)
    partial(basel_tel, Nat.0) = Real.0
    from_nat_one[Real]
    from_nat[Real](Nat.1) = Real.1
    Nat.0.suc = Nat.1
    from_nat[Real](Nat.0.suc) = Real.1
    basel_div_subst_den(Real.1, from_nat[Real](Nat.0.suc), Real.1)
    Real.1 / from_nat[Real](Nat.0.suc) = Real.1 / Real.1
    Real.1 / Real.1 = Real.1
    basel_sub_cong(Real.1, Real.1 / from_nat[Real](Nat.0.suc), Real.1)
    Real.1 - Real.1 / from_nat[Real](Nat.0.suc) = Real.1 - Real.1
    Real.1 - Real.1 = Real.0
    Real.1 - Real.1 / from_nat[Real](Nat.0.suc) = Real.0
    p(Nat.0)
    // step: partial(tel, j+1) = partial(tel, j) + tel(j) = 1 - 1/(j+2)
    forall(j: Nat) {
        if p(j) {
            partial_split_last(basel_tel, j)
            partial(basel_tel, j.suc) = partial(basel_tel, j) + basel_tel(j)
            p(j) = (partial(basel_tel, j) = Real.1 - Real.1 / from_nat[Real](j.suc))
            partial(basel_tel, j) = Real.1 - Real.1 / from_nat[Real](j.suc)
            basel_tel(j) = Real.1 / from_nat[Real](j.suc) - Real.1 / from_nat[Real](j.suc.suc)
            basel_add_subst2(partial(basel_tel, j), basel_tel(j),
                Real.1 - Real.1 / from_nat[Real](j.suc),
                Real.1 / from_nat[Real](j.suc) - Real.1 / from_nat[Real](j.suc.suc))
            partial(basel_tel, j.suc) =
                (Real.1 - Real.1 / from_nat[Real](j.suc)) +
                (Real.1 / from_nat[Real](j.suc) - Real.1 / from_nat[Real](j.suc.suc))
            basel_tel_cancel(Real.1 / from_nat[Real](j.suc), Real.1 / from_nat[Real](j.suc.suc))
            (Real.1 - Real.1 / from_nat[Real](j.suc)) +
                (Real.1 / from_nat[Real](j.suc) - Real.1 / from_nat[Real](j.suc.suc)) =
                Real.1 - Real.1 / from_nat[Real](j.suc.suc)
            partial(basel_tel, j.suc) = Real.1 - Real.1 / from_nat[Real](j.suc.suc)
            p(j.suc)
        }
    }
    p(Nat.0) and forall(j: Nat) { p(j) implies p(j.suc) }
    Nat.induction(p)
    p(n)
}

/// The partial sums of the telescoping majorant are at most one.
theorem basel_tel_partial_le_one(n: Nat) {
    partial(basel_tel, n) <= Real.1
} by {
    basel_tel_partial(n)
    partial(basel_tel, n) = Real.1 - Real.1 / from_nat[Real](n.suc)
    from_nat_suc_pos_real(n)
    from_nat[Real](n.suc) > Real.0
    basel_one_div_pos(from_nat[Real](n.suc))
    Real.1 / from_nat[Real](n.suc) > Real.0
    basel_neg_pos_le_zero(Real.1 / from_nat[Real](n.suc))
    -(Real.1 / from_nat[Real](n.suc)) <= Real.0
    lte_add_right(-(Real.1 / from_nat[Real](n.suc)), Real.0, Real.1)
    -(Real.1 / from_nat[Real](n.suc)) + Real.1 <= Real.0 + Real.1
    Real.0 + Real.1 = Real.1
    Real.1 - Real.1 / from_nat[Real](n.suc) <= Real.1
    partial(basel_tel, n) <= Real.1
}

/// Taking inverses reverses a strict inequality between positive reals.
theorem basel_inverse_antitone_strict(a: Real, b: Real) {
    a < b and a > Real.0 implies b.inverse < a.inverse
} by {
    if a < b and a > Real.0 {
        Real.0 < a
        lt_trans(Real.0, a, b)
        Real.0 < b
        inverse_on_positive_flips_inequality[Real](a, b)
        b.inverse < a.inverse
    }
}

/// Taking reciprocals reverses a non-strict inequality between positive
/// reals.
theorem basel_recip_antitone_pos(a: Real, b: Real) {
    a > Real.0 and a <= b implies Real.1 / b <= Real.1 / a
} by {
    if a > Real.0 and a <= b {
        lt_or_eq_of_lte[Real](a, b)
        a < b or a = b
        if a < b {
            basel_inverse_antitone_strict(a, b)
            b.inverse < a.inverse
            Real.1 / b = b.inverse
            Real.1 / a = a.inverse
            lte_of_lt[Real](b.inverse, a.inverse)
            b.inverse <= a.inverse
            basel_lte_subst2(Real.1 / b, Real.1 / a, b.inverse, a.inverse)
            Real.1 / b <= Real.1 / a
        } else {
            a = b
            basel_div_subst_den(Real.1, b, a)
            Real.1 / b = Real.1 / a
            Real.1 / b <= Real.1 / a
        }
    }
}

/// The difference of unit fractions is the quotient of the denominator
/// difference over the product.
theorem basel_recip_diff(a: Real, b: Real) {
    a != Real.0 and b != Real.0
    implies Real.1 / a - Real.1 / b = (b - a) / (a * b)
} by {
    if a != Real.0 and b != Real.0 {
        mul_div_cancel(b, a)
        b / (a * b) = Real.1 / a
        Real.1 / a = b / (a * b)
        mul_div_cancel(a, b)
        a / (b * a) = Real.1 / b
        b * a = a * b
        a / (a * b) = Real.1 / b
        Real.1 / b = a / (a * b)
        real_no_zero_divisors(a, b)
        a * b != Real.0
        b / (a * b) - a / (a * b) = (b - a) / (a * b)
        Real.1 / a - Real.1 / b = (b - a) / (a * b)
    }
}

/// A squared reciprocal is at most twice the telescoping difference:
/// `1 / a^2 <= 2 (1 / a - 1 / b)` for `1 <= a` and `b = a + 1`.
theorem basel_recip_sq_le_twice_tel(a: Real, b: Real) {
    a > Real.0 and b > Real.0 and b = a + Real.1 and Real.1 <= a
    implies Real.1 / (a * a) <= (Real.1 + Real.1) * (Real.1 / a - Real.1 / b)
} by {
    if a > Real.0 and b > Real.0 and b = a + Real.1 and Real.1 <= a {
        // the first denominator is at most the second: a <= b
        basel_zero_le_one
        Real.0 <= Real.1
        lte_add_right(Real.0, Real.1, a)
        Real.0 + a <= Real.1 + a
        Real.0 + a = a
        Real.1 + a = a + Real.1
        a <= a + Real.1
        b = a + Real.1
        a <= b
        // 1 / b <= 1 / a
        basel_recip_antitone_pos(a, b)
        Real.1 / b <= Real.1 / a
        // the second denominator is at most twice the first: b <= a + a
        lte_add_right(Real.1, a, a)
        Real.1 + a <= a + a
        b = a + Real.1
        b <= a + a
        // a * b <= 2 * (a * a)
        mul_le_mul_pos_right(b, a + a, a)
        b * a <= (a + a) * a
        b * a = a * b
        (a + a) * a = (Real.1 + Real.1) * (a * a)
        a * b <= (Real.1 + Real.1) * (a * a)
        // reciprocals reverse the ordering of the positive products
        a.is_positive
        b.is_positive
        mul_pos_pos(a, b)
        (a * b).is_positive
        mul_pos_pos(Real.1 + Real.1, a * a)
        ((Real.1 + Real.1) * (a * a)).is_positive
        basel_recip_antitone_pos(a * b, (Real.1 + Real.1) * (a * a))
        Real.1 / ((Real.1 + Real.1) * (a * a)) <= Real.1 / (a * b)
        // multiply by two on both sides
        two_positive
        (Real.1 + Real.1) > Real.0
        mul_le_mul_pos_left(Real.1 / ((Real.1 + Real.1) * (a * a)),
            Real.1 / (a * b), Real.1 + Real.1)
        (Real.1 + Real.1) * (Real.1 / ((Real.1 + Real.1) * (a * a))) <= (Real.1 + Real.1) * (Real.1 / (a * b))
        // the left-hand side is 1 / (a * a)
        (Real.1 + Real.1) * (a * a) = (a * a) * (Real.1 + Real.1)
        (Real.1 + Real.1) / ((Real.1 + Real.1) * (a * a)) =
            (Real.1 + Real.1) / ((a * a) * (Real.1 + Real.1))
        (Real.1 + Real.1) > Real.0
        (Real.1 + Real.1) != Real.0
        a != Real.0
        mul_not_zero[Real](a, a)
        a * a != Real.0
        div_mul_cancel_right(Real.1 + Real.1, a * a)
        (Real.1 + Real.1) / ((a * a) * (Real.1 + Real.1)) = Real.1 / (a * a)
        (Real.1 + Real.1) * (Real.1 / ((Real.1 + Real.1) * (a * a))) = Real.1 / (a * a)
        // the difference of unit fractions is the reciprocal of the product
        b = a + Real.1
        (a + Real.1) - a = Real.1
        b - a = Real.1
        basel_recip_diff(a, b)
        Real.1 / a - Real.1 / b = (b - a) / (a * b)
        basel_div_subst(b - a, Real.1, a * b)
        (b - a) / (a * b) = Real.1 / (a * b)
        Real.1 / (a * b) = Real.1 / a - Real.1 / b
        (Real.1 + Real.1) * (Real.1 / (a * b)) =
            (Real.1 + Real.1) * (Real.1 / a - Real.1 / b)
        Real.1 / (a * a) <= (Real.1 + Real.1) * (Real.1 / a - Real.1 / b)
    }
}

/// Each Basel term is at most twice its telescoping majorant.
theorem basel_term_le_twice_tel(k: Nat) {
    basel_term(k) <= (Real.1 + Real.1) * basel_tel(k)
} by {
    from_nat_suc_pos_real(k)
    from_nat[Real](k.suc) > Real.0
    from_nat_suc_pos_real(k.suc)
    from_nat[Real](k.suc.suc) > Real.0
    from_nat_add[Real](k.suc, Nat.1)
    from_nat[Real](k.suc + Nat.1) = from_nat[Real](k.suc) + from_nat[Real](Nat.1)
    from_nat_one[Real]
    from_nat[Real](Nat.1) = Real.1
    k.suc + Nat.1 = k.suc.suc
    from_nat[Real](k.suc.suc) = from_nat[Real](k.suc) + Real.1
    // 1 <= from_nat(k.suc)
    Nat.1 <= k.suc
    from_nat_lte_mono(Nat.1, k.suc)
    from_nat[Real](Nat.1) <= from_nat[Real](k.suc)
    from_nat_one[Real]
    from_nat[Real](Nat.1) = Real.1
    Real.1 <= from_nat[Real](k.suc)
    basel_recip_sq_le_twice_tel(from_nat[Real](k.suc), from_nat[Real](k.suc.suc))
    Real.1 / (from_nat[Real](k.suc) * from_nat[Real](k.suc)) <= (Real.1 + Real.1) * (Real.1 / from_nat[Real](k.suc) - Real.1 / from_nat[Real](k.suc.suc))
    basel_term(k) <= (Real.1 + Real.1) * basel_tel(k)
}

/// The partial sums of the Basel series are bounded above by two.
theorem basel_partial_le_two(n: Nat) {
    partial(basel_term, n) <= Real.1 + Real.1
} by {
    forall(k: Nat) {
        basel_term_le_twice_tel(k)
        basel_term(k) <= (Real.1 + Real.1) * basel_tel(k)
    }
    seq_lte(basel_term, basel_two_tel) =
        forall(k: Nat) { basel_term(k) <= basel_two_tel(k) }
    seq_lte(basel_term, basel_two_tel)
    partial_seq_lte(basel_term, basel_two_tel)
    seq_lte(partial(basel_term), partial(basel_two_tel))
    partial(basel_term, n) <= partial(basel_two_tel, n)
    basel_partial_two_tel(n)
    partial(basel_two_tel, n) = (Real.1 + Real.1) * partial(basel_tel, n)
    partial(basel_term, n) <= (Real.1 + Real.1) * partial(basel_tel, n)
    basel_tel_partial_le_one(n)
    partial(basel_tel, n) <= Real.1
    two_positive
    (Real.1 + Real.1) > Real.0
    mul_le_mul_pos_left(partial(basel_tel, n), Real.1, Real.1 + Real.1)
    (Real.1 + Real.1) * partial(basel_tel, n) <= (Real.1 + Real.1) * Real.1
    (Real.1 + Real.1) * Real.1 = Real.1 + Real.1
    lte_trans(partial(basel_term, n), (Real.1 + Real.1) * partial(basel_tel, n), Real.1 + Real.1)
    partial(basel_term, n) <= Real.1 + Real.1
}

/// Nonnegative terms have increasing partial sums.
theorem partial_nonneg_increasing(f: Nat -> Real) {
    is_lower_bound(f, Real.0) implies is_increasing(partial(f))
} by {
    if is_lower_bound(f, Real.0) {
        is_lower_bound(f, Real.0) = forall(k: Nat) { Real.0 <= f(k) }
        forall(n: Nat) {
            partial_split_last(f, n)
            partial(f, n.suc) = partial(f, n) + f(n)
            Real.0 <= f(n)
            lte_add_left(f(n), Real.0, partial(f, n))
            partial(f, n) + Real.0 <= partial(f, n) + f(n)
            partial(f, n) + Real.0 = partial(f, n)
            partial(f, n) <= partial(f, n.suc)
        }
        forall(n: Nat) { partial(f, n) <= partial(f, n.suc) }
        is_increasing(partial(f))
    }
}

/// The Basel series converges: the partial sums of the squared reciprocals
/// form a bounded increasing sequence.
///
/// This is the convergence half of the Basel problem; the value of the
/// limit (`pi^2 / 6`) is not yet in the library.
theorem theorems1000_basel_converges {
    converges(partial(basel_term))
} by {
    forall(k: Nat) {
        basel_term_nonneg(k)
        Real.0 <= basel_term(k)
    }
    is_lower_bound(basel_term, Real.0)
    partial_nonneg_increasing(basel_term)
    is_increasing(partial(basel_term))
    forall(n: Nat) {
        basel_partial_le_two(n)
        partial(basel_term, n) <= Real.1 + Real.1
    }
    is_upper_bound(partial(basel_term), Real.1 + Real.1)
    monotone_convergence_principle(partial(basel_term), Real.1 + Real.1)
    converges(partial(basel_term))
}
