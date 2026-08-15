// The mean value theorem (Joseph-Louis Lagrange, 1797; often attributed
// to Augustin-Louis Cauchy, 1823, who gave the modern proof via Rolle's
// theorem): if `f` is continuous on the closed interval `[a, b]` and
// differentiable on the open interval `(a, b)`, then for some strictly
// interior point `c`,
//     f'(c) = (f(b) - f(a)) / (b - a),
// i.e. the derivative equals the slope of the secant through the two
// endpoint values.  Geometrically, some tangent line is parallel to the
// chord joining the endpoints of the graph.
//
// The theorem is the central result of differential calculus: from it one
// derives the monotonicity criteria (a function with a nonnegative
// derivative is nondecreasing), the constancy of functions whose
// derivative vanishes (see the corollary sketched below), Taylor's
// theorem with remainder, l'Hôpital's rule, and the fundamental theorem
// of calculus; it is also the source of the error estimates in numerical
// analysis.  Cauchy's mean value theorem (the version with two functions)
// is the form used in the proofs of l'Hôpital's rule.
//
// The formal statement below uses the library's calculus vocabulary: the
// pointwise derivative predicate `has_derivative_at(f, c, d)` says that
// `d` is the derivative of `f` at `c` (`real/derivative_basic.ac`), the
// chord slope is `secant_slope(f, a, b)` (`real/mean_value.ac`), and
// `is_derivative_fn(f, df)` says that `df` is a pointwise derivative of
// `f` everywhere (`real/calculus_api.ac`).
//
// Proof sketch (the classical argument, as in the library's own proof of
// `mean_value_theorem` in `real/mean_value.ac`): apply Rolle's theorem to
// the secant-remainder function
//     g(x) = f(x) - s (x - a),   s = (f(b) - f(a)) / (b - a),
// which is continuous on `[a, b]`, differentiable on `(a, b)`, and
// satisfies `g(a) = g(b) = f(a)`.  Rolle's theorem gives an interior
// point `c` with `g'(c) = 0`, and since `g'(x) = f'(x) - s` this is
// `f'(c) = s`.
//
// Classical corollary (a function with zero derivative is constant): if
// `f' = 0` everywhere on an interval, the mean value theorem applied to
// any two points `a < x` of the interval gives `0 = f'(c) = (f(x) - f(a)) / (x - a)`,
// hence `f(x) = f(a)`; the library's `zero_derivative_imp_constant` in
// `real/mean_value.ac` proves exactly this.  The corollary is recorded
// below without proof, since exposing it through the interface is
// deferred (the interface is at its declaration limit).
//
// The wrapper below restates the library theorem `mean_value_theorem` of
// `real/mean_value.ac`, which is exposed through the public `real`
// interface together with the chord-slope helper `secant_slope`; the
// library proof is cited directly.

from real import Real, continuous, is_derivative_fn, has_derivative_at, secant_slope,
    mean_value_theorem

numerals Real

/// The mean value theorem: a continuous function on a closed interval
/// that is differentiable everywhere has an interior point whose
/// derivative equals the slope of the secant through the endpoint values.
theorem theorems1000_mean_value(f: Real -> Real, df: Real -> Real, a: Real, b: Real) {
    continuous(f) and a < b and is_derivative_fn(f, df)
    implies exists(c: Real) {
        a < c and c < b and has_derivative_at(f, c, secant_slope(f, a, b))
    }
} by {
    if continuous(f) and a < b and is_derivative_fn(f, df) {
        mean_value_theorem(f, df, a, b)
        exists(c: Real) { a < c and c < b and has_derivative_at(f, c, secant_slope(f, a, b)) }
    }
}

// theorem theorems1000_zero_derivative_constant(
//     f: Real -> Real, a: Real, b: Real, x: Real
// ) {
//     continuous(f) and a < b and is_derivative_fn(f, constant[Real, Real](Real.0)) and
//     closed_interval_set(a, b).contains(x)
//     implies f(x) = f(a)
// }
