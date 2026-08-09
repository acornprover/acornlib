from nat import Nat
from list import List, map, product, is_permutation, permutation_preserves_product
from number_theory.interface import congr_mod_mul, congr_mod_refl, congr_mod_symm,
    cancel_coprime, coprime_residues, coprime_residues_length, mul_mod_fn,
    mul_mod_residues, mul_mod_residues_is_permutation, product_coprime_residues_coprime,
    product_map_cons, product_map_scalar, scalar_mul_fn

numerals Nat

// Euler's totient theorem, the generalization of Fermat's little theorem
// proved by Leonhard Euler in 1763: if `a` is coprime to `n`, then
//     a^φ(n) ≡ 1  (mod n),
// where `φ(n)` (Euler's totient) counts the natural numbers in `[1, n]`
// coprime to `n` — the size of the multiplicative group of units modulo `n`.
// At a prime `n = p`, `φ(p) = p - 1` and the theorem reduces to Fermat's
// little theorem (see theorem_fermat_little.ac in this directory).
//
// Proof (the product argument): let `R` be the list of reduced residues
// modulo `n` (the numbers in `[0, n)` coprime to `n`, so `|R| = φ(n)`), and
// let `P` be their product.  Multiplication by the unit `a` permutes `R`
// modulo `n` (`mul_mod_residues_is_permutation`), so the product of the
// residues `(a * r) mod n` is again `P`.  On the other hand each factor
// `(a * r) mod n` is congruent to `a * r` modulo `n`, so the products are
// congruent:
//     P ≡ ∏_r (a r mod n) ≡ ∏_r (a r) = a^φ(n) · P  (mod n).
// Cancelling the factor `P` — itself coprime to `n`, since every reduced
// residue is — leaves `a^φ(n) ≡ 1 (mod n)`.
//
// The related statements on the divisibility of `φ` and on the value of the
// totient at prime powers are recorded below as commented theorems; the
// proofs need the multiplicative-order theory and the inclusion-exclusion
// counting inside `number_theory`, which are not part of the public
// interface.

/// Helper: multiplication modulo `n` preserves the product modulo `n`:
/// `∏_r (a r mod n) ≡ ∏_r (a r) (mod n)`.
theorem product_map_mul_mod_congr(n: Nat, a: Nat, l: List[Nat]) {
    product[Nat](map[Nat, Nat](l, mul_mod_fn(n, a))).congr_mod(
        product[Nat](map[Nat, Nat](l, scalar_mul_fn(a))),
        n
    )
} by {
    define p(x: List[Nat]) -> Bool {
        product[Nat](map[Nat, Nat](x, mul_mod_fn(n, a))).congr_mod(
            product[Nat](map[Nat, Nat](x, scalar_mul_fn(a))),
            n
        )
    }
    map[Nat, Nat](List.nil[Nat], mul_mod_fn(n, a)) = List.nil[Nat]
    map[Nat, Nat](List.nil[Nat], scalar_mul_fn(a)) = List.nil[Nat]
    product[Nat](List.nil[Nat]) = Nat.1
    congr_mod_refl(Nat.1, n)
    p(List.nil[Nat])
    forall(head: Nat, tail: List[Nat]) {
        if p(tail) {
            mul_mod_fn(n, a)(head).congr_mod(scalar_mul_fn(a)(head), n)
            product[Nat](map[Nat, Nat](tail, mul_mod_fn(n, a))).congr_mod(
                product[Nat](map[Nat, Nat](tail, scalar_mul_fn(a))),
                n
            )
            congr_mod_mul(
                mul_mod_fn(n, a)(head),
                scalar_mul_fn(a)(head),
                product[Nat](map[Nat, Nat](tail, mul_mod_fn(n, a))),
                product[Nat](map[Nat, Nat](tail, scalar_mul_fn(a))),
                n
            )
            (
                mul_mod_fn(n, a)(head) *
                product[Nat](map[Nat, Nat](tail, mul_mod_fn(n, a)))
            ).congr_mod(
                scalar_mul_fn(a)(head) *
                product[Nat](map[Nat, Nat](tail, scalar_mul_fn(a))),
                n
            )
            product_map_cons(mul_mod_fn(n, a), head, tail)
            product_map_cons(scalar_mul_fn(a), head, tail)
            p(List.cons(head, tail))
        }
    }
    forall(head: Nat, tail: List[Nat]) {
        p(tail) implies p(List.cons(head, tail))
    }
    p(List.nil[Nat]) and forall(head: Nat, tail: List[Nat]) {
        p(tail) implies p(List.cons(head, tail))
    }
    List.induction(function(ls: List[Nat]) { p(ls) })
    forall(ls: List[Nat]) { p(ls) }
    p(l)
}

/// Euler's theorem: `a^φ(n) ≡ 1 (mod n)` for `a` coprime to `n`.
theorem theorems1000_euler_theorem(n: Nat, a: Nat) {
    n != Nat.0 and a.coprime(n) implies a.pow(n.totient).congr_mod(Nat.1, n)
} by {
    if n != Nat.0 and a.coprime(n) {
        let r: List[Nat] = coprime_residues(n)
        let pr: Nat = product[Nat](r)
        product_coprime_residues_coprime(n)
        pr.coprime(n)
        coprime_residues_length(n)
        r.length = n.totient
        mul_mod_residues_is_permutation(n, a)
        is_permutation[Nat](mul_mod_residues(n, a), r)
        permutation_preserves_product(mul_mod_residues(n, a), r)
        product[Nat](mul_mod_residues(n, a)) = pr
        mul_mod_residues(n, a) = map[Nat, Nat](r, mul_mod_fn(n, a))
        product[Nat](map[Nat, Nat](r, mul_mod_fn(n, a))) = pr
        product_map_mul_mod_congr(n, a, r)
        product[Nat](map[Nat, Nat](r, mul_mod_fn(n, a))).congr_mod(
            product[Nat](map[Nat, Nat](r, scalar_mul_fn(a))),
            n
        )
        pr.congr_mod(product[Nat](map[Nat, Nat](r, scalar_mul_fn(a))), n)
        product_map_scalar(a, r)
        product[Nat](map[Nat, Nat](r, scalar_mul_fn(a))) = a.pow(r.length) * pr
        pr.congr_mod(a.pow(r.length) * pr, n)
        congr_mod_symm(pr, a.pow(r.length) * pr, n)
        (a.pow(r.length) * pr).congr_mod(pr, n)
        a.pow(r.length) * pr = pr * a.pow(r.length)
        pr = pr * Nat.1
        (pr * a.pow(r.length)).congr_mod(pr * Nat.1, n)
        r.length = n.totient
        (pr * a.pow(n.totient)).congr_mod(pr * Nat.1, n)
        cancel_coprime(pr, n, a.pow(n.totient), Nat.1)
        a.pow(n.totient).congr_mod(Nat.1, n)
    }
}

// ---------------------------------------------------------------------------
// Related totient statements (stated, not proved in this file).
//
// The following three theorems are classical companions of Euler's theorem.
// Their proofs live inside the `number_theory` package (the multiplicative
// order theory in `number_theory/multiplicative_order.ac` and the counting
// arguments in `number_theory/totient.ac`) and are not part of the public
// interface, so they are recorded here as statements only.
//
// Euler's theorem on the divisibility of `φ`: since `a^φ(n) ≡ 1 (mod n)` and
// the multiplicative order `ord_n(a)` of `a` is the least positive exponent
// with this property, the order divides every such exponent, in particular
// `ord_n(a) | φ(n)`.  (Library: `multiplicative_order_mod_divides_totient`.)
//
//     theorem theorems1000_order_divides_totient(a: Nat, n: Nat) {
//         n != Nat.0 and a.coprime(n) implies
//             multiplicative_order_mod(a, n).divides(n.totient)
//     }
//
// The totient of a prime is `φ(p) = p - 1`: among `0, 1, ..., p - 1` only
// `0` fails to be coprime to `p`.  (Library: `totient_prime`.)
//
//     theorem theorems1000_totient_prime(p: Nat) {
//         p.is_prime implies p.totient = p - Nat.1
//     }
//
// The totient of a prime power is `φ(p^(k+1)) = p^(k+1) - p^k = (p-1) p^k`:
// among `0, ..., p^(k+1) - 1` exactly the `p^k` multiples of `p` fail to be
// coprime to `p^(k+1)`.  Together with multiplicativity on coprime factors
// (`totient_mul_coprime_positive`, in the public interface) and the Chinese
// remainder theorem, this yields the multiplicative formula
// `φ(n) = n ∏_{p | n} (1 - 1/p)`.  (Library: `totient_p_pow` and
// `totient_p_pow_factored`.)
//
//     theorem theorems1000_totient_prime_pow(p: Nat, k: Nat) {
//         p.is_prime implies (p.pow(k.suc)).totient = p.pow(k.suc) - p.pow(k)
//     }
// ---------------------------------------------------------------------------
