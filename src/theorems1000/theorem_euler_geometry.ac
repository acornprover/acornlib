from pair import Pair
from real import Real
from top100 import r2_sub, r2_norm_sq
from theorems1000.r2_helpers import r2_cross

// Euler's theorem in geometry: in a triangle, the distance `OI` between the
// circumcentre `O` and the incentre `I` satisfies
//     OI^2 = R^2 - 2 R r,
// where `R` is the circumradius and `r` the inradius.  In particular
// `R >= 2 r` (Euler's inequality).
//
// The triangle has vertices `a`, `b`, `c`, circumcentre `o` and circumradius
// witness `R` (`R^2 = |a-o|^2 = |b-o|^2 = |c-o|^2`).  The incentre `i` and
// inradius witness `r` are given through the squared distance to a side
// line, `dist(i, line(u, v))^2 = cross(i - u, v - u)^2 / |v-u|^2`:
// `r^2 * |b-a|^2 = cross(i - a, b - a)^2` and cyclically.  The conclusion
// is `|i - o|^2 = R^2 - 2 R r`.
//
// The statement is commented out because the proof needs the standard power
// of a point argument together with the two elementary identities
// `AI = r / (A/2).sin` and `IM = 2 R (A/2).sin` for the second intersection
// `M` of the angle bisector `AI` with the circumcircle; these require the
// sine law and the angle-bisector distance formula, which are not yet in
// the library.  The sketch is recorded below.
//
// Proof sketch: let `M` be the second intersection of the internal angle
// bisector from `A` with the circumcircle (the midpoint of the arc `BC`).
// The power of `I` with respect to the circumcircle is `OI^2 - R^2`; along
// the secant `AM` the same power is `-AI * IM` (with the sign of an
// interior point), so `R^2 - OI^2 = AI * IM`.  Now `AI = r / (A/2).sin`
// (the inradius is `AI` times the sine of half the angle at `A`), and since
// `M` is the arc midpoint, `IM = MB = 2 R (A/2).sin` by the sine law in the
// triangle `AMB` (the angle `MAB` subtends the arc `MB` of measure `A`).
// Hence `AI * IM = 2 R r` and `R^2 - OI^2 = 2 R r`, i.e.
// `OI^2 = R^2 - 2 R r`.
//
// theorem theorems1000_euler_geometry(
//     o: Pair[Real, Real], rad: Real,
//     a: Pair[Real, Real], b: Pair[Real, Real], c: Pair[Real, Real],
//     i: Pair[Real, Real], r: Real
// ) {
//     rad * rad = r2_norm_sq(r2_sub(a, o)) and
//     rad * rad = r2_norm_sq(r2_sub(b, o)) and
//     rad * rad = r2_norm_sq(r2_sub(c, o)) and
//     r * r * r2_norm_sq(r2_sub(b, a)) = r2_cross(r2_sub(i, a), r2_sub(b, a)) * r2_cross(r2_sub(i, a), r2_sub(b, a)) and
//     r * r * r2_norm_sq(r2_sub(c, b)) = r2_cross(r2_sub(i, b), r2_sub(c, b)) * r2_cross(r2_sub(i, b), r2_sub(c, b)) and
//     r * r * r2_norm_sq(r2_sub(a, c)) = r2_cross(r2_sub(i, c), r2_sub(a, c)) * r2_cross(r2_sub(i, c), r2_sub(a, c))
//     implies
//     r2_norm_sq(r2_sub(i, o)) = rad * rad - (Real.1 + Real.1) * rad * r
// }
