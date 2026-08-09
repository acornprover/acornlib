from nat import Nat
from number_theory.interface import prime_imp_wilson_factorial_congr,
    wilson_factorial_congr_imp_prime

numerals Nat

// Wilson's theorem (Joseph-Louis Lagrange, 1771; published by Edward
// Waring 1770): a natural number `p > 1` is prime exactly when
//     (p - 1)! ≡ -1  (mod p).
// Written with natural numbers, `p - 1` is the residue class of `-1` modulo
// `p`, so the congruence is `(p - 1)! ≡ p - 1 (mod p)`.
//
// The forward direction is the congruence half of the theorem: for a prime
// `p`, pairing each reduced residue `2 <= x <= p - 2` with its modular
// inverse (which is a different residue in that range, since `x^2 = 1`
// forces `x = 1` or `x = p - 1`) makes the product of `1, 2, ..., p - 1`
// congruent to `1 * (p - 1) * (pairs, each congruent to 1)`, i.e. to
// `p - 1`.  This is the library theorem `prime_imp_wilson_factorial_congr`.
//
// The converse direction is the primality half: if `(p - 1)! ≡ p - 1
// (mod p)` and `p != 1`, then `p` has no proper divisor `1 < d < p` (any
// such divisor would divide `(p - 1)!`, but the congruence says `p` divides
// `(p - 1)! - (p - 1)`, and `p - 1` is coprime to `p`, a contradiction), so
// `p` is prime.  This is the library theorem
// `wilson_factorial_congr_imp_prime`.
//
// Wilson's theorem characterizes primes by a single factorial congruence and
// is the basis of Gauss's proof of the law of quadratic reciprocity.

/// Wilson's theorem, prime to congruence: `(p - 1)! ≡ p - 1 (mod p)`.
theorem theorems1000_wilson_prime_imp_congr(p: Nat) {
    p.is_prime implies (p - Nat.1).factorial.congr_mod(p - Nat.1, p)
} by {
    if p.is_prime {
        prime_imp_wilson_factorial_congr(p)
        (p - Nat.1).factorial.congr_mod(p - Nat.1, p)
    }
}

/// Wilson's theorem, congruence to prime: the factorial congruence forces
/// primality (for `p != 1`).
theorem theorems1000_wilson_congr_imp_prime(p: Nat) {
    p != Nat.1 and (p - Nat.1).factorial.congr_mod(p - Nat.1, p) implies p.is_prime
} by {
    if p != Nat.1 and (p - Nat.1).factorial.congr_mod(p - Nat.1, p) {
        wilson_factorial_congr_imp_prime(p)
        p.is_prime
    }
}
