from nat import Nat
from real import Real, converges_to, subsequence, is_unbounded, bolzano_weierstrass
from order import is_monotone

numerals Real

// The Bolzano–Weierstrass theorem (Bernard Bolzano, 1817; Karl
// Weierstrass, 1860s): every bounded sequence of real numbers has a
// convergent subsequence.  Equivalently, every sequence of points of a
// closed bounded interval (or of a compact subset of R^n) has a
// subsequence converging to a point of that set.  The theorem is one of
// the two standard formulations of the completeness of the real numbers
// (the other being the monotone convergence theorem, see
// `theorem_monotone_convergence.ac`): it is equivalent to the Heine–Borel
// theorem via the identification of compactness with sequential
// compactness, and it underlies the existence of the Riemann integral, the
// extreme value theorem, and Peano's existence theorem for ordinary
// differential equations.
//
// The formal statement below encodes the subsequence by an index map
// `f: Nat -> Nat` that is monotone and unbounded (the library's notion of
// a subsequence index; `subsequence(a, f, n) = compose(a, f)(n)`), and
// encodes boundedness of the sequence by two reals `lo <= a(n) <= hi`
// bounding every term.  The conclusion is that some such `f` and some real
// `l` satisfy `converges_to(subsequence(a, f), l)`.
//
// Proof: the library proves the theorem as `bolzano_weierstrass` in
// `real/bolzano_weierstrass.ac`, by the monotone-subsequence (peaks)
// argument: every real sequence has a nondecreasing or nonincreasing
// subsequence (if peaks occur beyond every index the successive peaks form
// a nonincreasing subsequence; otherwise, past the last peak every index
// is followed by a strictly larger value and the least such successors form
// a strictly increasing subsequence), and a monotone subsequence of a
// bounded sequence converges by the monotone convergence principle.  The
// library statement is identical to the one below, so the proof is a direct
// citation.

/// The Bolzano–Weierstrass theorem: every bounded real sequence has a
/// convergent subsequence.
theorem theorems1000_bolzano_weierstrass(a: Nat -> Real) {
    exists(lo: Real, hi: Real) {
        forall(n: Nat) { lo <= a(n) and a(n) <= hi }
    } implies
        exists(f: Nat -> Nat, l: Real) {
            is_monotone(f) and is_unbounded(f) and
            converges_to(subsequence(a, f), l)
        }
} by {
    if exists(lo: Real, hi: Real) {
        forall(n: Nat) { lo <= a(n) and a(n) <= hi }
    } {
        bolzano_weierstrass(a)
        exists(f: Nat -> Nat, l: Real) {
            is_monotone(f) and is_unbounded(f) and
            converges_to(subsequence(a, f), l)
        }
    }
}
