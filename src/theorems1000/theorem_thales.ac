from pair import Pair
from real import Real
from top100 import r2_add, r2_neg, r2_sub, r2_dot, r2_norm_sq
from theorems1000.r2_helpers import r2_sub_through, r2_sub_reverse_neg, r2_midpoint_sub_swap, r2_dot_sub_add, r2_add_comm, r2_sub_eq_add_neg

// Thales's theorem: the angle in a semicircle is a right angle.
// If `a` and `b` are the endpoints of a diameter of a circle with center
// `center`, and `c` lies on the same circle, then the angle at `c` subtended
// by the diameter is right: the vectors `c - a` and `c - b` are orthogonal.

/// The endpoints of the diameter are opposite: from `a + b = 2c`, the
/// displacement from the center to `b` is the negation of the displacement
/// from the center to `a`.
theorem thales_diameter_opposite(a: Pair[Real, Real], b: Pair[Real, Real], center: Pair[Real, Real]) {
    r2_add(a, b) = r2_add(center, center) implies
    r2_sub(b, center) = r2_neg(r2_sub(a, center))
} by {
    if r2_add(a, b) = r2_add(center, center) {
        r2_midpoint_sub_swap(a, b, center)
        r2_sub(center, b) = r2_sub(a, center)
        r2_sub_reverse_neg(b, center)
        r2_sub(b, center) = r2_neg(r2_sub(center, b))
        r2_sub(b, center) = r2_neg(r2_sub(a, center))
    }
}

/// A point on the circle with diameter endpoints `a`, `b` sees the diameter
/// under a right angle: `(c - a) . (c - b) = 0`.
///
/// The circle has center `center` and squared radius `radius_sq`; the points
/// `a`, `b` and `c` lie on it, and `a + b = 2 * center` says the center is
/// the midpoint of the segment `ab`, so `ab` is a diameter.
theorem theorems1000_thales(
    center: Pair[Real, Real], radius_sq: Real,
    a: Pair[Real, Real], b: Pair[Real, Real], c: Pair[Real, Real]
) {
    r2_norm_sq(r2_sub(a, center)) = radius_sq and
    r2_norm_sq(r2_sub(b, center)) = radius_sq and
    r2_norm_sq(r2_sub(c, center)) = radius_sq and
    r2_add(a, b) = r2_add(center, center)
    implies r2_dot(r2_sub(c, a), r2_sub(c, b)) = Real.0
} by {
    if r2_norm_sq(r2_sub(a, center)) = radius_sq and
        r2_norm_sq(r2_sub(b, center)) = radius_sq and
        r2_norm_sq(r2_sub(c, center)) = radius_sq and
        r2_add(a, b) = r2_add(center, center) {
        r2_norm_sq(r2_sub(a, center)) = radius_sq
        r2_norm_sq(r2_sub(c, center)) = radius_sq
        // v = a - center is the reflection of the diameter half towards `b`.
        thales_diameter_opposite(a, b, center)
        r2_sub(b, center) = r2_neg(r2_sub(a, center))
        // c - a = (c - center) - (a - center)
        r2_sub_through(a, center, c)
        r2_sub(c, a) = r2_add(r2_sub(center, a), r2_sub(c, center))
        r2_sub_reverse_neg(a, center)
        r2_sub(center, a) = r2_neg(r2_sub(a, center))
        r2_sub(c, a) = r2_add(r2_neg(r2_sub(a, center)), r2_sub(c, center))
        r2_add_comm(r2_neg(r2_sub(a, center)), r2_sub(c, center))
        r2_sub(c, a) = r2_add(r2_sub(c, center), r2_neg(r2_sub(a, center)))
        // c - b = (c - center) + (a - center)
        r2_sub_through(b, center, c)
        r2_sub(c, b) = r2_add(r2_sub(center, b), r2_sub(c, center))
        r2_sub(center, b) = r2_sub(a, center)
        r2_sub(c, b) = r2_add(r2_sub(a, center), r2_sub(c, center))
        r2_add_comm(r2_sub(a, center), r2_sub(c, center))
        r2_sub(c, b) = r2_add(r2_sub(c, center), r2_sub(a, center))
        // (u - v) . (u + v) = |u|^2 - |v|^2 with u = c - center, v = a - center
        r2_sub_eq_add_neg(r2_sub(c, center), r2_sub(a, center))
        r2_sub(r2_sub(c, center), r2_sub(a, center)) = r2_add(r2_sub(c, center), r2_neg(r2_sub(a, center)))
        r2_sub(c, a) = r2_sub(r2_sub(c, center), r2_sub(a, center))
        r2_sub(c, b) = r2_add(r2_sub(c, center), r2_sub(a, center))
        r2_dot(r2_sub(c, a), r2_sub(c, b)) =
            r2_dot(r2_sub(r2_sub(c, center), r2_sub(a, center)), r2_add(r2_sub(c, center), r2_sub(a, center)))
        r2_dot_sub_add(r2_sub(c, center), r2_sub(a, center))
        r2_dot(r2_sub(r2_sub(c, center), r2_sub(a, center)), r2_add(r2_sub(c, center), r2_sub(a, center))) =
            r2_norm_sq(r2_sub(c, center)) - r2_norm_sq(r2_sub(a, center))
        r2_norm_sq(r2_sub(c, center)) - r2_norm_sq(r2_sub(a, center)) = radius_sq - radius_sq
        radius_sq - radius_sq = Real.0
        r2_dot(r2_sub(r2_sub(c, center), r2_sub(a, center)), r2_add(r2_sub(c, center), r2_sub(a, center))) = Real.0
        r2_dot(r2_sub(c, a), r2_sub(c, b)) = Real.0
    }
}
