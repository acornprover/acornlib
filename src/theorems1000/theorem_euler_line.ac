from pair import Pair
from real import Real
from top100 import r2_add, r2_sub, r2_dot, r2_norm_sq
from theorems1000.r2_helpers import r2_sub_through, r2_add_comm, r2_add_three_rotate,
    r2_dot_comm, r2_dot_sub_add, r2_sub_add_pair_distrib, r2_sub_sub_swap,
    r2_sub_sub_same_base, r2_add_sub_left_cancel, r2_sub_add_right_cancel,
    r2_triple_sub_distrib, r2_triple_sub_right

// Euler's theorem in geometry (the Euler line): in a triangle, the centroid,
// the circumcenter, and the orthocenter are collinear, and the orthocenter
// is three times as far from the circumcenter as the centroid (the centroid
// divides the segment from the circumcenter to the orthocenter in the ratio
// 1 : 2).  With `o` the circumcenter, `h = a + b + c - 2o` the candidate
// orthocenter and `g` the centroid (`3g = a + b + c`), the theorem states
// that `h` lies on all three altitudes and that `h - o = 3 (g - o)`.

/// The candidate orthocenter lies on the altitude from `c` to the side `ab`.
///
/// With `h = a + b + c - 2o` and `a`, `b` on the circle centered at `o`, the
/// altitude through `c` is perpendicular to `ab`: `(h - c) . (a - b) = 0`.
theorem euler_altitude_dot(
    o: Pair[Real, Real], radius_sq: Real,
    a: Pair[Real, Real], b: Pair[Real, Real], c: Pair[Real, Real],
    h: Pair[Real, Real]
) {
    r2_norm_sq(r2_sub(a, o)) = radius_sq and
    r2_norm_sq(r2_sub(b, o)) = radius_sq and
    r2_sub(r2_add(r2_add(a, b), c), r2_add(o, o)) = h
    implies
    r2_dot(r2_sub(h, c), r2_sub(a, b)) = Real.0
} by {
    if r2_norm_sq(r2_sub(a, o)) = radius_sq and
        r2_norm_sq(r2_sub(b, o)) = radius_sq and
        r2_sub(r2_add(r2_add(a, b), c), r2_add(o, o)) = h {
        r2_norm_sq(r2_sub(a, o)) = radius_sq
        r2_norm_sq(r2_sub(b, o)) = radius_sq
        r2_sub(r2_add(r2_add(a, b), c), r2_add(o, o)) = h
        // h - c = (a - o) + (b - o)
        r2_sub_sub_swap(r2_add(r2_add(a, b), c), r2_add(o, o), c)
        r2_sub(r2_sub(r2_add(r2_add(a, b), c), r2_add(o, o)), c) =
            r2_sub(r2_sub(r2_add(r2_add(a, b), c), c), r2_add(o, o))
        r2_sub_through(c, r2_add(a, b), r2_add(r2_add(a, b), c))
        r2_sub(r2_add(r2_add(a, b), c), c) =
            r2_add(r2_sub(r2_add(a, b), c), r2_sub(r2_add(r2_add(a, b), c), r2_add(a, b)))
        r2_add_sub_left_cancel(r2_add(a, b), c)
        r2_sub(r2_add(r2_add(a, b), c), r2_add(a, b)) = c
        r2_sub_add_right_cancel(r2_add(a, b), c)
        r2_add(r2_sub(r2_add(a, b), c), c) = r2_add(a, b)
        r2_sub(r2_add(r2_add(a, b), c), c) = r2_add(a, b)
        r2_sub(r2_sub(r2_add(r2_add(a, b), c), c), r2_add(o, o)) = r2_sub(r2_add(a, b), r2_add(o, o))
        r2_sub_add_pair_distrib(a, b, o, o)
        r2_sub(r2_add(a, b), r2_add(o, o)) = r2_add(r2_sub(a, o), r2_sub(b, o))
        r2_sub(r2_sub(r2_add(r2_add(a, b), c), r2_add(o, o)), c) =
            r2_add(r2_sub(a, o), r2_sub(b, o))
        r2_sub(h, c) = r2_add(r2_sub(a, o), r2_sub(b, o))
        // a - b = (a - o) - (b - o)
        r2_sub_sub_same_base(o, b, a)
        r2_sub(r2_sub(a, o), r2_sub(b, o)) = r2_sub(a, b)
        // (u + v) . (u - v) = |u|^2 - |v|^2 with u = a - o, v = b - o
        r2_dot_comm(r2_add(r2_sub(a, o), r2_sub(b, o)), r2_sub(r2_sub(a, o), r2_sub(b, o)))
        r2_dot(r2_add(r2_sub(a, o), r2_sub(b, o)), r2_sub(r2_sub(a, o), r2_sub(b, o))) =
            r2_dot(r2_sub(r2_sub(a, o), r2_sub(b, o)), r2_add(r2_sub(a, o), r2_sub(b, o)))
        r2_dot_sub_add(r2_sub(a, o), r2_sub(b, o))
        r2_dot(r2_sub(r2_sub(a, o), r2_sub(b, o)), r2_add(r2_sub(a, o), r2_sub(b, o))) =
            r2_norm_sq(r2_sub(a, o)) - r2_norm_sq(r2_sub(b, o))
        r2_norm_sq(r2_sub(a, o)) - r2_norm_sq(r2_sub(b, o)) = radius_sq - radius_sq
        radius_sq - radius_sq = Real.0
        r2_dot(r2_add(r2_sub(a, o), r2_sub(b, o)), r2_sub(r2_sub(a, o), r2_sub(b, o))) = Real.0
        r2_dot(r2_sub(h, c), r2_sub(a, b)) = Real.0
    }
}

/// Euler's theorem in geometry: the Euler line.
///
/// For a triangle with vertices `a`, `b`, `c`, circumcenter `o`, candidate
/// orthocenter `h = a + b + c - 2o` and centroid `g` (with `3g = a + b + c`),
/// the point `h` lies on all three altitudes, and the centroid, circumcenter
/// and orthocenter are collinear with `h - o = 3 (g - o)`.
theorem theorems1000_euler_line(
    o: Pair[Real, Real], radius_sq: Real,
    a: Pair[Real, Real], b: Pair[Real, Real], c: Pair[Real, Real],
    g: Pair[Real, Real], h: Pair[Real, Real]
) {
    r2_norm_sq(r2_sub(a, o)) = radius_sq and
    r2_norm_sq(r2_sub(b, o)) = radius_sq and
    r2_norm_sq(r2_sub(c, o)) = radius_sq and
    r2_sub(r2_add(r2_add(a, b), c), r2_add(o, o)) = h and
    r2_add(g, r2_add(g, g)) = r2_add(r2_add(a, b), c)
    implies
    r2_dot(r2_sub(h, a), r2_sub(b, c)) = Real.0 and
    r2_dot(r2_sub(h, b), r2_sub(c, a)) = Real.0 and
    r2_dot(r2_sub(h, c), r2_sub(a, b)) = Real.0 and
    r2_add(r2_sub(g, o), r2_add(r2_sub(g, o), r2_sub(g, o))) = r2_sub(h, o)
} by {
    if r2_norm_sq(r2_sub(a, o)) = radius_sq and
        r2_norm_sq(r2_sub(b, o)) = radius_sq and
        r2_norm_sq(r2_sub(c, o)) = radius_sq and
        r2_sub(r2_add(r2_add(a, b), c), r2_add(o, o)) = h and
        r2_add(g, r2_add(g, g)) = r2_add(r2_add(a, b), c) {
        r2_norm_sq(r2_sub(a, o)) = radius_sq
        r2_norm_sq(r2_sub(b, o)) = radius_sq
        r2_norm_sq(r2_sub(c, o)) = radius_sq
        r2_sub(r2_add(r2_add(a, b), c), r2_add(o, o)) = h
        r2_add(g, r2_add(g, g)) = r2_add(r2_add(a, b), c)
        // h lies on the altitude from a, from b, and from c
        r2_add_three_rotate(a, b, c)
        r2_add(r2_add(a, b), c) = r2_add(r2_add(b, c), a)
        r2_sub(r2_add(r2_add(b, c), a), r2_add(o, o)) = r2_sub(r2_add(r2_add(a, b), c), r2_add(o, o))
        r2_sub(r2_add(r2_add(b, c), a), r2_add(o, o)) = h
        euler_altitude_dot(o, radius_sq, b, c, a, h)
        r2_dot(r2_sub(h, a), r2_sub(b, c)) = Real.0
        r2_add_three_rotate(c, a, b)
        r2_add(r2_add(c, a), b) = r2_add(r2_add(a, b), c)
        r2_sub(r2_add(r2_add(c, a), b), r2_add(o, o)) = r2_sub(r2_add(r2_add(a, b), c), r2_add(o, o))
        r2_sub(r2_add(r2_add(c, a), b), r2_add(o, o)) = h
        euler_altitude_dot(o, radius_sq, c, a, b, h)
        r2_dot(r2_sub(h, b), r2_sub(c, a)) = Real.0
        euler_altitude_dot(o, radius_sq, a, b, c, h)
        r2_dot(r2_sub(h, c), r2_sub(a, b)) = Real.0
        // 3(g - o) = (g + g + g) - (o + o + o) = (a + b + c) - 3o
        r2_triple_sub_distrib(g, o)
        r2_add(r2_sub(g, o), r2_add(r2_sub(g, o), r2_sub(g, o))) =
            r2_sub(r2_add(g, r2_add(g, g)), r2_add(o, r2_add(o, o)))
        r2_sub(r2_add(g, r2_add(g, g)), r2_add(o, r2_add(o, o))) =
            r2_sub(r2_add(r2_add(a, b), c), r2_add(o, r2_add(o, o)))
        r2_add(r2_sub(g, o), r2_add(r2_sub(g, o), r2_sub(g, o))) =
            r2_sub(r2_add(r2_add(a, b), c), r2_add(o, r2_add(o, o)))
        // h - o = ((a + b + c) - 2o) - o = (a + b + c) - 3o
        r2_triple_sub_right(r2_add(r2_add(a, b), c), o)
        r2_sub(r2_sub(r2_add(r2_add(a, b), c), r2_add(o, o)), o) =
            r2_sub(r2_add(r2_add(a, b), c), r2_add(o, r2_add(o, o)))
        r2_sub(h, o) = r2_sub(r2_sub(r2_add(r2_add(a, b), c), r2_add(o, o)), o)
        r2_sub(h, o) = r2_sub(r2_add(r2_add(a, b), c), r2_add(o, r2_add(o, o)))
        r2_add(r2_sub(g, o), r2_add(r2_sub(g, o), r2_sub(g, o))) = r2_sub(h, o)
        r2_dot(r2_sub(h, a), r2_sub(b, c)) = Real.0 and
            r2_dot(r2_sub(h, b), r2_sub(c, a)) = Real.0 and
            r2_dot(r2_sub(h, c), r2_sub(a, b)) = Real.0 and
            r2_add(r2_sub(g, o), r2_add(r2_sub(g, o), r2_sub(g, o))) = r2_sub(h, o)
    }
}
