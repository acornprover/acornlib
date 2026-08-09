from pair import Pair
from real import Real, one_half_plus_one_half
from top100 import r2_add, r2_neg, r2_sub, r2_dot, r2_norm_sq
from theorems1000.r2_helpers import r2_smul, r2_rot90, r2_rot90_sub, r2_rot90_norm_sq,
    r2_dot_rot90_self, r2_dot_smul_right, r2_dot_add_right, r2_norm_sq_smul,
    r2_norm_sq_add_expansion, r2_norm_sq_sub_expansion, r2_norm_sq_neg,
    r2_sub_eq_add_neg, r2_sub_through, r2_add_neg_right, r2_neg_add, r2_neg_neg,
    r2_add_assoc, r2_add_comm, r2_add_sub_rearrange, r2_sub_add_right_same,
    r2_sub_add_move_left, r2_sub_add_move, r2_sub_reverse_neg, r2_sub_sub_same_base,
    r2_smul_add_distrib, r2_smul_sub_distrib, r2_sub_add_pair_distrib,
    real_half_half_four, real_half_twice

// Pompeiu's theorem: for an equilateral triangle `abc` and an arbitrary
// point `p` in the plane, the three distances `|p - a|`, `|p - b|`,
// `|p - c|` are the side lengths of a triangle.
//
// The equilateral triangle is recorded through a formal square root of
// three, the real parameter `s` with `s * s = 3` (the same device as in
// the Napoleon theorem): the rotation by 60 degrees is
//     R60(v) = (v + s J v) / 2
// where `J` is the quarter turn, and the equilateral condition is
// `c - a = R60(b - a)`.
//
// The triangle with the required side lengths is obtained by the classical
// rotation construction: rotate `p` about `a` by 60 degrees to the point
// `p' = a + R60(p - a)`.  Then the triangle with vertices `p`, `p'`, `c`
// has side lengths
//     |p - p'| = |p - a|,   |p' - c| = |p - b|,   |p - c| = |p - c|:
// the first because a rotation preserves norms and the chord of a 60-degree
// rotation has the same length as the rotated vector
// (`|v - R60(v)|^2 = |v|^2`), the second because rotations are isometries:
// `p' - c = R60(p - a) - (c - a) = R60(p - a) - R60(b - a) = R60(p - b)`.
// The statement below asserts the existence of such a triangle, and the
// construction is carried out in the proof.

// ---------------------------------------------------------------------------
// The rotation by 60 degrees, with a formal square root of three.
// ---------------------------------------------------------------------------

/// The rotation by 60 degrees: `R60(v) = (v + s J v) / 2`.
define r2_rot60(s: Real, v: Pair[Real, Real]) -> Pair[Real, Real] {
    r2_smul(Real.one_half, r2_add(v, r2_smul(s, r2_rot90(v))))
}

/// A rotation preserves the squared norm: `|R60(v)|^2 = |v|^2` when
/// `s * s = 3`.
theorem pompeiu_rot60_norm_sq(s: Real, v: Pair[Real, Real]) {
    s * s = Real.1 + Real.1 + Real.1 implies
    r2_norm_sq(r2_rot60(s, v)) = r2_norm_sq(v)
} by {
    if s * s = Real.1 + Real.1 + Real.1 {
        s * s = Real.1 + Real.1 + Real.1
        r2_norm_sq_add_expansion(v, r2_smul(s, r2_rot90(v)))
        r2_norm_sq(r2_add(v, r2_smul(s, r2_rot90(v)))) =
            r2_norm_sq(v) + r2_norm_sq(r2_smul(s, r2_rot90(v))) +
            r2_dot(v, r2_smul(s, r2_rot90(v))) + r2_dot(v, r2_smul(s, r2_rot90(v)))
        r2_norm_sq_smul(s, r2_rot90(v))
        r2_norm_sq(r2_smul(s, r2_rot90(v))) = s * s * r2_norm_sq(r2_rot90(v))
        r2_rot90_norm_sq(v)
        r2_norm_sq(r2_rot90(v)) = r2_norm_sq(v)
        r2_norm_sq(r2_smul(s, r2_rot90(v))) = s * s * r2_norm_sq(v)
        r2_norm_sq(r2_smul(s, r2_rot90(v))) = (Real.1 + Real.1 + Real.1) * r2_norm_sq(v)
        r2_dot_smul_right(s, v, r2_rot90(v))
        r2_dot(v, r2_smul(s, r2_rot90(v))) = s * r2_dot(v, r2_rot90(v))
        r2_dot_rot90_self(v)
        r2_dot(v, r2_rot90(v)) = Real.0
        s * r2_dot(v, r2_rot90(v)) = Real.0
        r2_dot(v, r2_smul(s, r2_rot90(v))) = Real.0
        r2_norm_sq(r2_add(v, r2_smul(s, r2_rot90(v)))) =
            r2_norm_sq(v) + (Real.1 + Real.1 + Real.1) * r2_norm_sq(v) + Real.0 + Real.0
        r2_norm_sq(v) + (Real.1 + Real.1 + Real.1) * r2_norm_sq(v) + Real.0 + Real.0 =
            (Real.1 + Real.1 + Real.1 + Real.1) * r2_norm_sq(v)
        r2_norm_sq(r2_add(v, r2_smul(s, r2_rot90(v)))) =
            (Real.1 + Real.1 + Real.1 + Real.1) * r2_norm_sq(v)
        r2_norm_sq_smul(Real.one_half, r2_add(v, r2_smul(s, r2_rot90(v))))
        r2_norm_sq(r2_rot60(s, v)) =
            Real.one_half * Real.one_half * r2_norm_sq(r2_add(v, r2_smul(s, r2_rot90(v))))
        r2_norm_sq(r2_rot60(s, v)) =
            Real.one_half * Real.one_half * ((Real.1 + Real.1 + Real.1 + Real.1) * r2_norm_sq(v))
        real_half_half_four(r2_norm_sq(v))
        Real.one_half * Real.one_half * ((Real.1 + Real.1) * (Real.1 + Real.1)) * r2_norm_sq(v) =
            r2_norm_sq(v)
        Real.one_half * Real.one_half * ((Real.1 + Real.1 + Real.1 + Real.1) * r2_norm_sq(v)) =
            r2_norm_sq(v)
        r2_norm_sq(r2_rot60(s, v)) = r2_norm_sq(v)
    }
}

/// A rotation distributes over subtraction:
/// `R60(u - v) = R60(u) - R60(v)`.
theorem pompeiu_rot60_sub(s: Real, u: Pair[Real, Real], v: Pair[Real, Real]) {
    r2_rot60(s, r2_sub(u, v)) = r2_sub(r2_rot60(s, u), r2_rot60(s, v))
} by {
    r2_rot90_sub(u, v)
    r2_rot90(r2_sub(u, v)) = r2_sub(r2_rot90(u), r2_rot90(v))
    r2_smul_sub_distrib(s, r2_rot90(u), r2_rot90(v))
    r2_smul(s, r2_sub(r2_rot90(u), r2_rot90(v))) =
        r2_sub(r2_smul(s, r2_rot90(u)), r2_smul(s, r2_rot90(v)))
    r2_smul(s, r2_rot90(r2_sub(u, v))) =
        r2_sub(r2_smul(s, r2_rot90(u)), r2_smul(s, r2_rot90(v)))
    r2_add(r2_sub(u, v), r2_smul(s, r2_rot90(r2_sub(u, v)))) =
        r2_add(r2_sub(u, v), r2_sub(r2_smul(s, r2_rot90(u)), r2_smul(s, r2_rot90(v))))
    r2_sub_add_pair_distrib(u, r2_smul(s, r2_rot90(u)), v, r2_smul(s, r2_rot90(v)))
    r2_sub(r2_add(u, r2_smul(s, r2_rot90(u))), r2_add(v, r2_smul(s, r2_rot90(v)))) =
        r2_add(r2_sub(u, v), r2_sub(r2_smul(s, r2_rot90(u)), r2_smul(s, r2_rot90(v))))
    r2_add(r2_sub(u, v), r2_smul(s, r2_rot90(r2_sub(u, v)))) =
        r2_sub(r2_add(u, r2_smul(s, r2_rot90(u))), r2_add(v, r2_smul(s, r2_rot90(v))))
    r2_smul(Real.one_half, r2_add(r2_sub(u, v), r2_smul(s, r2_rot90(r2_sub(u, v))))) =
        r2_smul(Real.one_half,
            r2_sub(r2_add(u, r2_smul(s, r2_rot90(u))), r2_add(v, r2_smul(s, r2_rot90(v)))))
    r2_smul_sub_distrib(Real.one_half, r2_add(u, r2_smul(s, r2_rot90(u))),
        r2_add(v, r2_smul(s, r2_rot90(v))))
    r2_smul(Real.one_half, r2_sub(r2_add(u, r2_smul(s, r2_rot90(u))),
        r2_add(v, r2_smul(s, r2_rot90(v))))) =
        r2_sub(
            r2_smul(Real.one_half, r2_add(u, r2_smul(s, r2_rot90(u)))),
            r2_smul(Real.one_half, r2_add(v, r2_smul(s, r2_rot90(v)))))
    r2_smul(Real.one_half, r2_add(u, r2_smul(s, r2_rot90(u)))) = r2_rot60(s, u)
    r2_smul(Real.one_half, r2_add(v, r2_smul(s, r2_rot90(v)))) = r2_rot60(s, v)
    r2_smul(Real.one_half, r2_add(r2_sub(u, v), r2_smul(s, r2_rot90(r2_sub(u, v))))) =
        r2_sub(r2_rot60(s, u), r2_rot60(s, v))
    r2_rot60(s, r2_sub(u, v)) =
        r2_smul(Real.one_half, r2_add(r2_sub(u, v), r2_smul(s, r2_rot90(r2_sub(u, v)))))
    r2_rot60(s, r2_sub(u, v)) = r2_sub(r2_rot60(s, u), r2_rot60(s, v))
}

/// `v . R60(v) = |v|^2 / 2` when `s * s = 3`.
theorem pompeiu_rot60_dot(s: Real, v: Pair[Real, Real]) {
    s * s = Real.1 + Real.1 + Real.1 implies
    r2_dot(v, r2_rot60(s, v)) = Real.one_half * r2_norm_sq(v)
} by {
    if s * s = Real.1 + Real.1 + Real.1 {
        r2_dot_smul_right(Real.one_half, v, r2_add(v, r2_smul(s, r2_rot90(v))))
        r2_dot(v, r2_smul(Real.one_half, r2_add(v, r2_smul(s, r2_rot90(v))))) =
            Real.one_half * r2_dot(v, r2_add(v, r2_smul(s, r2_rot90(v))))
        r2_dot(v, r2_rot60(s, v)) = Real.one_half * r2_dot(v, r2_add(v, r2_smul(s, r2_rot90(v))))
        r2_dot_add_right(v, v, r2_smul(s, r2_rot90(v)))
        r2_dot(v, r2_add(v, r2_smul(s, r2_rot90(v)))) =
            r2_dot(v, v) + r2_dot(v, r2_smul(s, r2_rot90(v)))
        r2_norm_sq(v) = r2_dot(v, v)
        r2_dot(v, r2_add(v, r2_smul(s, r2_rot90(v)))) =
            r2_norm_sq(v) + r2_dot(v, r2_smul(s, r2_rot90(v)))
        r2_dot_smul_right(s, v, r2_rot90(v))
        r2_dot(v, r2_smul(s, r2_rot90(v))) = s * r2_dot(v, r2_rot90(v))
        r2_dot_rot90_self(v)
        r2_dot(v, r2_rot90(v)) = Real.0
        s * r2_dot(v, r2_rot90(v)) = Real.0
        r2_dot(v, r2_smul(s, r2_rot90(v))) = Real.0
        r2_dot(v, r2_add(v, r2_smul(s, r2_rot90(v)))) = r2_norm_sq(v) + Real.0
        r2_norm_sq(v) + Real.0 = r2_norm_sq(v)
        r2_dot(v, r2_add(v, r2_smul(s, r2_rot90(v)))) = r2_norm_sq(v)
        r2_dot(v, r2_rot60(s, v)) = Real.one_half * r2_norm_sq(v)
    }
}

// ---------------------------------------------------------------------------
// The two rotated sides of the constructed triangle.
// ---------------------------------------------------------------------------

/// `|p - (a + R60(p - a))|^2 = |p - a|^2`: the chord of the rotation has the
/// length of the rotated vector.
theorem pompeiu_xy_norm(s: Real, a: Pair[Real, Real], p: Pair[Real, Real]) {
    s * s = Real.1 + Real.1 + Real.1 implies
    r2_norm_sq(r2_sub(p, r2_add(a, r2_rot60(s, r2_sub(p, a))))) = r2_norm_sq(r2_sub(p, a))
} by {
    if s * s = Real.1 + Real.1 + Real.1 {
        s * s = Real.1 + Real.1 + Real.1
        r2_sub_add_move_left(p, a, r2_rot60(s, r2_sub(p, a)))
        r2_sub(p, r2_add(a, r2_rot60(s, r2_sub(p, a)))) =
            r2_sub(r2_sub(p, a), r2_rot60(s, r2_sub(p, a)))
        r2_norm_sq_sub_expansion(r2_sub(p, a), r2_rot60(s, r2_sub(p, a)))
        r2_norm_sq(r2_sub(r2_sub(p, a), r2_rot60(s, r2_sub(p, a)))) =
            r2_norm_sq(r2_sub(p, a)) + r2_norm_sq(r2_rot60(s, r2_sub(p, a))) -
            r2_dot(r2_sub(p, a), r2_rot60(s, r2_sub(p, a))) -
            r2_dot(r2_sub(p, a), r2_rot60(s, r2_sub(p, a)))
        r2_norm_sq(r2_sub(p, r2_add(a, r2_rot60(s, r2_sub(p, a))))) =
            r2_norm_sq(r2_sub(p, a)) + r2_norm_sq(r2_rot60(s, r2_sub(p, a))) -
            r2_dot(r2_sub(p, a), r2_rot60(s, r2_sub(p, a))) -
            r2_dot(r2_sub(p, a), r2_rot60(s, r2_sub(p, a)))
        pompeiu_rot60_norm_sq(s, r2_sub(p, a))
        r2_norm_sq(r2_rot60(s, r2_sub(p, a))) = r2_norm_sq(r2_sub(p, a))
        pompeiu_rot60_dot(s, r2_sub(p, a))
        r2_dot(r2_sub(p, a), r2_rot60(s, r2_sub(p, a))) =
            Real.one_half * r2_norm_sq(r2_sub(p, a))
        r2_norm_sq(r2_sub(p, r2_add(a, r2_rot60(s, r2_sub(p, a))))) =
            r2_norm_sq(r2_sub(p, a)) + r2_norm_sq(r2_sub(p, a)) -
            Real.one_half * r2_norm_sq(r2_sub(p, a)) - Real.one_half * r2_norm_sq(r2_sub(p, a))
        one_half_plus_one_half
        Real.one_half + Real.one_half = Real.1
        real_half_twice(r2_norm_sq(r2_sub(p, a)))
        Real.one_half * r2_norm_sq(r2_sub(p, a)) + Real.one_half * r2_norm_sq(r2_sub(p, a)) =
            r2_norm_sq(r2_sub(p, a))
        r2_norm_sq(r2_sub(p, a)) + r2_norm_sq(r2_sub(p, a)) -
            Real.one_half * r2_norm_sq(r2_sub(p, a)) - Real.one_half * r2_norm_sq(r2_sub(p, a)) =
            r2_norm_sq(r2_sub(p, a)) + r2_norm_sq(r2_sub(p, a)) - r2_norm_sq(r2_sub(p, a))
        r2_norm_sq(r2_sub(p, a)) + r2_norm_sq(r2_sub(p, a)) - r2_norm_sq(r2_sub(p, a)) =
            r2_norm_sq(r2_sub(p, a))
        r2_norm_sq(r2_sub(p, a)) + r2_norm_sq(r2_sub(p, a)) -
            Real.one_half * r2_norm_sq(r2_sub(p, a)) - Real.one_half * r2_norm_sq(r2_sub(p, a)) =
            r2_norm_sq(r2_sub(p, a))
        r2_norm_sq(r2_sub(p, r2_add(a, r2_rot60(s, r2_sub(p, a))))) = r2_norm_sq(r2_sub(p, a))
    }
}

/// `|(a + R60(p - a)) - c|^2 = |p - b|^2` for an equilateral triangle:
/// the image of the rotation is an isometry.
theorem pompeiu_yz_norm(
    s: Real, a: Pair[Real, Real], b: Pair[Real, Real],
    c: Pair[Real, Real], p: Pair[Real, Real]
) {
    s * s = Real.1 + Real.1 + Real.1 and
    r2_sub(c, a) = r2_rot60(s, r2_sub(b, a))
    implies
    r2_norm_sq(r2_sub(r2_add(a, r2_rot60(s, r2_sub(p, a))), c)) = r2_norm_sq(r2_sub(p, b))
} by {
    if s * s = Real.1 + Real.1 + Real.1 and r2_sub(c, a) = r2_rot60(s, r2_sub(b, a)) {
        s * s = Real.1 + Real.1 + Real.1
        r2_sub(c, a) = r2_rot60(s, r2_sub(b, a))
        r2_sub_add_move(a, r2_rot60(s, r2_sub(p, a)), c)
        r2_sub(r2_add(a, r2_rot60(s, r2_sub(p, a))), c) =
            r2_sub(r2_rot60(s, r2_sub(p, a)), r2_sub(c, a))
        r2_sub(r2_add(a, r2_rot60(s, r2_sub(p, a))), c) =
            r2_sub(r2_rot60(s, r2_sub(p, a)), r2_rot60(s, r2_sub(b, a)))
        pompeiu_rot60_sub(s, r2_sub(p, a), r2_sub(b, a))
        r2_rot60(s, r2_sub(r2_sub(p, a), r2_sub(b, a))) =
            r2_sub(r2_rot60(s, r2_sub(p, a)), r2_rot60(s, r2_sub(b, a)))
        r2_sub(r2_add(a, r2_rot60(s, r2_sub(p, a))), c) =
            r2_rot60(s, r2_sub(r2_sub(p, a), r2_sub(b, a)))
        r2_sub_sub_same_base(a, b, p)
        r2_sub(r2_sub(p, a), r2_sub(b, a)) = r2_sub(p, b)
        r2_sub(r2_add(a, r2_rot60(s, r2_sub(p, a))), c) = r2_rot60(s, r2_sub(p, b))
        r2_norm_sq(r2_sub(r2_add(a, r2_rot60(s, r2_sub(p, a))), c)) =
            r2_norm_sq(r2_rot60(s, r2_sub(p, b)))
        pompeiu_rot60_norm_sq(s, r2_sub(p, b))
        r2_norm_sq(r2_rot60(s, r2_sub(p, b))) = r2_norm_sq(r2_sub(p, b))
        r2_norm_sq(r2_sub(r2_add(a, r2_rot60(s, r2_sub(p, a))), c)) = r2_norm_sq(r2_sub(p, b))
    }
}

// ---------------------------------------------------------------------------
// Pompeiu's theorem.
// ---------------------------------------------------------------------------

/// Pompeiu's theorem: for an equilateral triangle `abc` and any point `p`,
/// the distances `|p - a|`, `|p - b|`, `|p - c|` are the side lengths of a
/// triangle.
///
/// The equilateral triangle is given by `c - a = R60(b - a)` with the
/// formal square root `s` of three, and the conclusion asserts the
/// existence of a triangle `x y z` with those side lengths; the proof takes
/// `x = p`, `y = a + R60(p - a)`, `z = c`.
theorem theorems1000_pompeiu(
    s: Real, a: Pair[Real, Real], b: Pair[Real, Real],
    c: Pair[Real, Real], p: Pair[Real, Real]
) {
    s * s = Real.1 + Real.1 + Real.1 and
    r2_sub(c, a) = r2_rot60(s, r2_sub(b, a))
    implies
    exists(x: Pair[Real, Real], y: Pair[Real, Real], z: Pair[Real, Real]) {
        r2_norm_sq(r2_sub(x, y)) = r2_norm_sq(r2_sub(p, a)) and
        r2_norm_sq(r2_sub(y, z)) = r2_norm_sq(r2_sub(p, b)) and
        r2_norm_sq(r2_sub(z, x)) = r2_norm_sq(r2_sub(p, c))
    }
} by {
    if s * s = Real.1 + Real.1 + Real.1 and r2_sub(c, a) = r2_rot60(s, r2_sub(b, a)) {
        s * s = Real.1 + Real.1 + Real.1
        r2_sub(c, a) = r2_rot60(s, r2_sub(b, a))
        let xw: Pair[Real, Real] = p
        let yw: Pair[Real, Real] = r2_add(a, r2_rot60(s, r2_sub(p, a)))
        let zw: Pair[Real, Real] = c
        // |xw - yw|^2 = |p - a|^2.
        pompeiu_xy_norm(s, a, p)
        r2_norm_sq(r2_sub(xw, yw)) = r2_norm_sq(r2_sub(p, a))
        // |yw - zw|^2 = |p - b|^2.
        pompeiu_yz_norm(s, a, b, c, p)
        r2_norm_sq(r2_sub(yw, zw)) = r2_norm_sq(r2_sub(p, b))
        // |zw - xw|^2 = |p - c|^2.
        r2_norm_sq(r2_sub(zw, xw)) = r2_norm_sq(r2_sub(c, p))
        r2_sub_reverse_neg(p, c)
        r2_sub(p, c) = r2_neg(r2_sub(c, p))
        r2_norm_sq(r2_sub(p, c)) = r2_norm_sq(r2_neg(r2_sub(c, p)))
        r2_norm_sq_neg(r2_sub(c, p))
        r2_norm_sq(r2_neg(r2_sub(c, p))) = r2_norm_sq(r2_sub(c, p))
        r2_norm_sq(r2_sub(p, c)) = r2_norm_sq(r2_sub(c, p))
        r2_norm_sq(r2_sub(zw, xw)) = r2_norm_sq(r2_sub(p, c))
        r2_norm_sq(r2_sub(xw, yw)) = r2_norm_sq(r2_sub(p, a)) and
            r2_norm_sq(r2_sub(yw, zw)) = r2_norm_sq(r2_sub(p, b)) and
            r2_norm_sq(r2_sub(zw, xw)) = r2_norm_sq(r2_sub(p, c))
        exists(x: Pair[Real, Real], y: Pair[Real, Real], z: Pair[Real, Real]) {
            r2_norm_sq(r2_sub(x, y)) = r2_norm_sq(r2_sub(p, a)) and
            r2_norm_sq(r2_sub(y, z)) = r2_norm_sq(r2_sub(p, b)) and
            r2_norm_sq(r2_sub(z, x)) = r2_norm_sq(r2_sub(p, c))
        }
    }
}
