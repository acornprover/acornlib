from pair import Pair
from real import Real
from top100 import r2_sub, r2_norm_sq
from theorems1000.r2_helpers import r2_cross

// Feuerbach's theorem: the nine-point circle of a triangle is tangent to
// the incircle and to each of the three excircles of the triangle.  This is
// one of the most celebrated results about the nine-point circle (whose
// side midpoints and Euler points are proved in
// `theorems1000_nine_point_circle` in this directory).
//
// The statement is recorded with the triangle `a b c`, circumcenter `o`,
// orthocenter `h = a + b + c - 2 o`, nine-point center `n` with
// `2 n = o + h`, and the incenter `i` with inradius witness `r`
// (the squared distance from `i` to each side line is `r^2`, written
// through the cross product as in `theorems1000_euler_geometry`).  The
// circumradius witness `rad` satisfies `rad^2 = |a - o|^2`.  Tangency of
// the nine-point circle (radius `rad / 2`) and the incircle (radius `r`)
// is the equality `4 |i - n|^2 = (rad - 2 r)^2` of squared distances
// between centers (internal tangency).
//
// Proof sketch: Feuerbach's original proof (1822) uses the power of the
// incenter with respect to the nine-point circle; a standard modern proof
// (Guinand 1984) computes the distance `NI` from the nine-point center to
// the incenter in terms of the circumradius `R`, the inradius `r` and the
// angles, obtaining `NI = R / 2 - r`; equivalently, with Euler's formula
// `OI^2 = R^2 - 2 R r` for the circumcenter-incenter distance and the
// Euler-line relation between `O`, `N`, and the de Longchamps point, the
// squared distance expands to `(R/2 - r)^2`.  The computation is a long
// trigonometric identity in the half-angles; the incenter distance formula
// and the sine law required for it are not yet in the library.
//
// theorem theorems1000_feuerbach(
//     o: Pair[Real, Real], rad: Real,
//     a: Pair[Real, Real], b: Pair[Real, Real], c: Pair[Real, Real],
//     h: Pair[Real, Real], n: Pair[Real, Real], i: Pair[Real, Real],
//     r: Real
// ) {
//     rad * rad = r2_norm_sq(r2_sub(a, o)) and
//     rad * rad = r2_norm_sq(r2_sub(b, o)) and
//     rad * rad = r2_norm_sq(r2_sub(c, o)) and
//     r2_sub(r2_add(r2_add(a, b), c), r2_add(o, o)) = h and
//     r2_add(o, h) = r2_add(n, n) and
//     r * r * r2_norm_sq(r2_sub(b, a)) =
//         r2_cross(r2_sub(i, a), r2_sub(b, a)) * r2_cross(r2_sub(i, a), r2_sub(b, a)) and
//     r * r * r2_norm_sq(r2_sub(c, b)) =
//         r2_cross(r2_sub(i, b), r2_sub(c, b)) * r2_cross(r2_sub(i, b), r2_sub(c, b)) and
//     r * r * r2_norm_sq(r2_sub(a, c)) =
//         r2_cross(r2_sub(i, c), r2_sub(a, c)) * r2_cross(r2_sub(i, c), r2_sub(a, c))
//     implies
//     (Real.1 + Real.1 + Real.1 + Real.1) * r2_norm_sq(r2_sub(i, n)) =
//         (rad - (Real.1 + Real.1) * r) * (rad - (Real.1 + Real.1) * r)
// }
