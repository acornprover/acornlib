from pair import Pair
from real import Real
from top100 import r2_add, r2_neg, r2_sub, r2_norm_sq
from theorems1000.r2_helpers import r2_sub_eq_add_neg, r2_sub_through, r2_add_neg_right,
    r2_add_zero_left, r2_add_zero_right, r2_add_assoc, r2_add_comm, r2_neg_add,
    r2_sub_sub_sub_pair, r2_norm_sq_neg, r2_sub_reverse_neg, r2_add_three_rotate,
    r2_sub_add_right_same, r2_sub_add_same_left, r2_sub_double

// Johnson's theorem: three circles of equal radius through a common point
// `p` meet again pairwise at three further points, and those three points
// lie on a circle of the same radius as the original circles.
//
// With the common point `p` and centers `o1`, `o2`, `o3` of the three
// circles, the circle with center `oi` and squared radius `rad` passes
// through `p` exactly when `rad = |oi - p|^2`.  The second intersection of
// the circles centered at `oi` and `oj` is the point
//     xij = oi + oj - p
// (the reflection of the common point `p` through the midpoint of the
// centers), which lies on both circles:
//     |xij - oi|^2 = |oj - p|^2 = rad and |xij - oj|^2 = |oi - p|^2 = rad.
// The three second-intersection points `x12`, `x23`, `x31` all lie on the
// circle of radius `rad` centered at
//     q = o1 + o2 + o3 - 2p,
// since `x12 - q = p - o3` and cyclically, so
//     |x12 - q|^2 = |o3 - p|^2 = rad
// and cyclically.  This is the claim proved below.

// ---------------------------------------------------------------------------
// The second-intersection displacement.
// ---------------------------------------------------------------------------

/// For `x = o1 + o2 - p` and `q = o1 + o2 + o3 - 2p`,
/// `x - q = p - o3`.
theorem johnson_displacement(
    o1: Pair[Real, Real], o2: Pair[Real, Real], o3: Pair[Real, Real],
    p: Pair[Real, Real], x: Pair[Real, Real], q: Pair[Real, Real]
) {
    x = r2_add(r2_add(o1, o2), r2_neg(p)) and
    q = r2_sub(r2_add(r2_add(o1, o2), o3), r2_add(p, p))
    implies
    r2_sub(x, q) = r2_sub(p, o3)
} by {
    if x = r2_add(r2_add(o1, o2), r2_neg(p)) and
        q = r2_sub(r2_add(r2_add(o1, o2), o3), r2_add(p, p)) {
        x = r2_add(r2_add(o1, o2), r2_neg(p))
        q = r2_sub(r2_add(r2_add(o1, o2), o3), r2_add(p, p))
        r2_sub_eq_add_neg(r2_add(o1, o2), p)
        r2_sub(r2_add(o1, o2), p) = r2_add(r2_add(o1, o2), r2_neg(p))
        r2_sub(r2_add(o1, o2), p) = x
        r2_sub(x, q) = r2_sub(r2_sub(r2_add(o1, o2), p), q)
        r2_sub_sub_sub_pair(r2_add(o1, o2), p, r2_add(r2_add(o1, o2), o3), r2_add(p, p))
        r2_sub(r2_sub(r2_add(o1, o2), p), r2_sub(r2_add(r2_add(o1, o2), o3), r2_add(p, p))) =
            r2_add(
                r2_sub(r2_add(o1, o2), r2_add(r2_add(o1, o2), o3)),
                r2_sub(r2_add(p, p), p))
        r2_sub(x, q) =
            r2_add(
                r2_sub(r2_add(o1, o2), r2_add(r2_add(o1, o2), o3)),
                r2_sub(r2_add(p, p), p))
        r2_sub_add_same_left(r2_add(o1, o2), o3)
        r2_sub(r2_add(o1, o2), r2_add(r2_add(o1, o2), o3)) = r2_neg(o3)
        r2_sub_double(p)
        r2_sub(r2_add(p, p), p) = p
        r2_sub(x, q) = r2_add(r2_neg(o3), p)
        r2_sub_eq_add_neg(p, o3)
        r2_sub(p, o3) = r2_add(p, r2_neg(o3))
        r2_add_comm(r2_neg(o3), p)
        r2_add(r2_neg(o3), p) = r2_add(p, r2_neg(o3))
        r2_add(r2_neg(o3), p) = r2_sub(p, o3)
        r2_sub(x, q) = r2_sub(p, o3)
    }
}

// ---------------------------------------------------------------------------
// Faithfulness: the second-intersection point lies on both circles.
// ---------------------------------------------------------------------------

/// The point `x12 = o1 + o2 - p` lies on the circle centered at `o1` of
/// radius squared `rad` (which passes through `p`).
theorem johnson_x12_on_circle_one(
    o1: Pair[Real, Real], o2: Pair[Real, Real], p: Pair[Real, Real],
    rad: Real, x12: Pair[Real, Real]
) {
    x12 = r2_sub(r2_add(o1, o2), p) and rad = r2_norm_sq(r2_sub(o2, p))
    implies
    r2_norm_sq(r2_sub(x12, o1)) = rad
} by {
    if x12 = r2_sub(r2_add(o1, o2), p) and rad = r2_norm_sq(r2_sub(o2, p)) {
        x12 = r2_sub(r2_add(o1, o2), p)
        rad = r2_norm_sq(r2_sub(o2, p))
        r2_sub_through(o1, r2_add(o1, o2), x12)
        r2_sub(x12, o1) = r2_add(r2_sub(r2_add(o1, o2), o1), r2_sub(x12, r2_add(o1, o2)))
        r2_sub_add_right_same(o1, o2)
        r2_sub(r2_add(o1, o2), o1) = o2
        r2_sub_eq_add_neg(r2_add(o1, o2), p)
        r2_sub(r2_add(o1, o2), p) = r2_add(r2_add(o1, o2), r2_neg(p))
        r2_sub(r2_add(o1, o2), p) = x12
        r2_sub_add_right_same(r2_add(o1, o2), r2_neg(p))
        r2_sub(r2_add(r2_add(o1, o2), r2_neg(p)), r2_add(o1, o2)) = r2_neg(p)
        r2_sub(x12, r2_add(o1, o2)) = r2_neg(p)
        r2_sub(x12, o1) = r2_add(o2, r2_neg(p))
        r2_sub_eq_add_neg(o2, p)
        r2_sub(o2, p) = r2_add(o2, r2_neg(p))
        r2_sub(x12, o1) = r2_sub(o2, p)
        r2_norm_sq(r2_sub(x12, o1)) = r2_norm_sq(r2_sub(o2, p))
        rad = r2_norm_sq(r2_sub(o2, p))
        r2_norm_sq(r2_sub(o2, p)) = rad
        r2_norm_sq(r2_sub(x12, o1)) = rad
    }
}

/// The point `x12 = o1 + o2 - p` lies on the circle centered at `o2` of
/// radius squared `rad` (which passes through `p`).
theorem johnson_x12_on_circle_two(
    o1: Pair[Real, Real], o2: Pair[Real, Real], p: Pair[Real, Real],
    rad: Real, x12: Pair[Real, Real]
) {
    x12 = r2_sub(r2_add(o1, o2), p) and rad = r2_norm_sq(r2_sub(o1, p))
    implies
    r2_norm_sq(r2_sub(x12, o2)) = rad
} by {
    if x12 = r2_sub(r2_add(o1, o2), p) and rad = r2_norm_sq(r2_sub(o1, p)) {
        x12 = r2_sub(r2_add(o1, o2), p)
        rad = r2_norm_sq(r2_sub(o1, p))
        r2_sub_through(o2, r2_add(o1, o2), x12)
        r2_sub(x12, o2) = r2_add(r2_sub(r2_add(o1, o2), o2), r2_sub(x12, r2_add(o1, o2)))
        r2_add_comm(o1, o2)
        r2_add(o1, o2) = r2_add(o2, o1)
        r2_sub_add_right_same(o2, o1)
        r2_sub(r2_add(o2, o1), o2) = o1
        r2_sub(r2_add(o1, o2), o2) = o1
        r2_sub_eq_add_neg(r2_add(o1, o2), p)
        r2_sub(r2_add(o1, o2), p) = r2_add(r2_add(o1, o2), r2_neg(p))
        r2_sub(r2_add(o1, o2), p) = x12
        r2_sub_add_right_same(r2_add(o1, o2), r2_neg(p))
        r2_sub(r2_add(r2_add(o1, o2), r2_neg(p)), r2_add(o1, o2)) = r2_neg(p)
        r2_sub(x12, r2_add(o1, o2)) = r2_neg(p)
        r2_sub(x12, o2) = r2_add(o1, r2_neg(p))
        r2_sub_eq_add_neg(o1, p)
        r2_sub(o1, p) = r2_add(o1, r2_neg(p))
        r2_sub(x12, o2) = r2_sub(o1, p)
        r2_norm_sq(r2_sub(x12, o2)) = r2_norm_sq(r2_sub(o1, p))
        rad = r2_norm_sq(r2_sub(o1, p))
        r2_norm_sq(r2_sub(o1, p)) = rad
        r2_norm_sq(r2_sub(x12, o2)) = rad
    }
}

// ---------------------------------------------------------------------------
// Johnson's theorem.
// ---------------------------------------------------------------------------

/// Johnson's theorem: if three circles of equal radius pass through a common
/// point `p`, then their three pairwise second intersections lie on a circle
/// of the same radius.
///
/// The circles have centers `o1`, `o2`, `o3` and common squared radius
/// `rad` (`rad = |oi - p|^2`), the second intersections are the points
/// `x12 = o1 + o2 - p`, `x23 = o2 + o3 - p`, `x31 = o3 + o1 - p`, and the
/// conclusion is that these three points are concyclic with radius `rad`.
theorem theorems1000_johnson(
    o1: Pair[Real, Real], o2: Pair[Real, Real], o3: Pair[Real, Real],
    p: Pair[Real, Real], rad: Real,
    x12: Pair[Real, Real], x23: Pair[Real, Real], x31: Pair[Real, Real]
) {
    rad = r2_norm_sq(r2_sub(o1, p)) and
    rad = r2_norm_sq(r2_sub(o2, p)) and
    rad = r2_norm_sq(r2_sub(o3, p)) and
    x12 = r2_sub(r2_add(o1, o2), p) and
    x23 = r2_sub(r2_add(o2, o3), p) and
    x31 = r2_sub(r2_add(o3, o1), p)
    implies
    exists(c: Pair[Real, Real]) {
        r2_norm_sq(r2_sub(x12, c)) = rad and
        r2_norm_sq(r2_sub(x23, c)) = rad and
        r2_norm_sq(r2_sub(x31, c)) = rad
    }
} by {
    if rad = r2_norm_sq(r2_sub(o1, p)) and
        rad = r2_norm_sq(r2_sub(o2, p)) and
        rad = r2_norm_sq(r2_sub(o3, p)) and
        x12 = r2_sub(r2_add(o1, o2), p) and
        x23 = r2_sub(r2_add(o2, o3), p) and
        x31 = r2_sub(r2_add(o3, o1), p) {
        rad = r2_norm_sq(r2_sub(o1, p))
        rad = r2_norm_sq(r2_sub(o2, p))
        rad = r2_norm_sq(r2_sub(o3, p))
        x12 = r2_sub(r2_add(o1, o2), p)
        x23 = r2_sub(r2_add(o2, o3), p)
        x31 = r2_sub(r2_add(o3, o1), p)
        r2_sub_eq_add_neg(r2_add(o1, o2), p)
        r2_sub(r2_add(o1, o2), p) = r2_add(r2_add(o1, o2), r2_neg(p))
        r2_sub_eq_add_neg(r2_add(o2, o3), p)
        r2_sub(r2_add(o2, o3), p) = r2_add(r2_add(o2, o3), r2_neg(p))
        r2_sub_eq_add_neg(r2_add(o3, o1), p)
        r2_sub(r2_add(o3, o1), p) = r2_add(r2_add(o3, o1), r2_neg(p))
        // The center of the Johnson circle: q = o1 + o2 + o3 - 2p.
        let q: Pair[Real, Real] = r2_sub(r2_add(r2_add(o1, o2), o3), r2_add(p, p))
        // |x12 - q|^2 = |p - o3|^2 = rad.
        johnson_displacement(o1, o2, o3, p, x12, q)
        r2_sub(x12, q) = r2_sub(p, o3)
        r2_norm_sq(r2_sub(x12, q)) = r2_norm_sq(r2_sub(p, o3))
        r2_sub_reverse_neg(p, o3)
        r2_sub(p, o3) = r2_neg(r2_sub(o3, p))
        r2_norm_sq(r2_sub(p, o3)) = r2_norm_sq(r2_neg(r2_sub(o3, p)))
        r2_norm_sq_neg(r2_sub(o3, p))
        r2_norm_sq(r2_neg(r2_sub(o3, p))) = r2_norm_sq(r2_sub(o3, p))
        r2_norm_sq(r2_sub(p, o3)) = r2_norm_sq(r2_sub(o3, p))
        rad = r2_norm_sq(r2_sub(o3, p))
        r2_norm_sq(r2_sub(o3, p)) = rad
        r2_norm_sq(r2_sub(p, o3)) = rad
        r2_norm_sq(r2_sub(x12, q)) = rad
        // |x23 - q|^2 = |p - o1|^2 = rad.
        r2_add_three_rotate(o1, o2, o3)
        r2_add(r2_add(o1, o2), o3) = r2_add(r2_add(o2, o3), o1)
        r2_sub(r2_add(r2_add(o1, o2), o3), r2_add(p, p)) =
            r2_sub(r2_add(r2_add(o2, o3), o1), r2_add(p, p))
        q = r2_sub(r2_add(r2_add(o2, o3), o1), r2_add(p, p))
        johnson_displacement(o2, o3, o1, p, x23, q)
        r2_sub(x23, q) = r2_sub(p, o1)
        r2_norm_sq(r2_sub(x23, q)) = r2_norm_sq(r2_sub(p, o1))
        r2_sub_reverse_neg(p, o1)
        r2_sub(p, o1) = r2_neg(r2_sub(o1, p))
        r2_norm_sq(r2_sub(p, o1)) = r2_norm_sq(r2_neg(r2_sub(o1, p)))
        r2_norm_sq_neg(r2_sub(o1, p))
        r2_norm_sq(r2_neg(r2_sub(o1, p))) = r2_norm_sq(r2_sub(o1, p))
        r2_norm_sq(r2_sub(p, o1)) = r2_norm_sq(r2_sub(o1, p))
        rad = r2_norm_sq(r2_sub(o1, p))
        r2_norm_sq(r2_sub(o1, p)) = rad
        r2_norm_sq(r2_sub(p, o1)) = rad
        r2_norm_sq(r2_sub(x23, q)) = rad
        // |x31 - q|^2 = |p - o2|^2 = rad.
        r2_add_three_rotate(o3, o1, o2)
        r2_add(r2_add(o3, o1), o2) = r2_add(r2_add(o1, o2), o3)
        r2_sub(r2_add(r2_add(o1, o2), o3), r2_add(p, p)) =
            r2_sub(r2_add(r2_add(o3, o1), o2), r2_add(p, p))
        q = r2_sub(r2_add(r2_add(o3, o1), o2), r2_add(p, p))
        johnson_displacement(o3, o1, o2, p, x31, q)
        r2_sub(x31, q) = r2_sub(p, o2)
        r2_norm_sq(r2_sub(x31, q)) = r2_norm_sq(r2_sub(p, o2))
        r2_sub_reverse_neg(p, o2)
        r2_sub(p, o2) = r2_neg(r2_sub(o2, p))
        r2_norm_sq(r2_sub(p, o2)) = r2_norm_sq(r2_neg(r2_sub(o2, p)))
        r2_norm_sq_neg(r2_sub(o2, p))
        r2_norm_sq(r2_neg(r2_sub(o2, p))) = r2_norm_sq(r2_sub(o2, p))
        r2_norm_sq(r2_sub(p, o2)) = r2_norm_sq(r2_sub(o2, p))
        rad = r2_norm_sq(r2_sub(o2, p))
        r2_norm_sq(r2_sub(o2, p)) = rad
        r2_norm_sq(r2_sub(p, o2)) = rad
        r2_norm_sq(r2_sub(x31, q)) = rad
        r2_norm_sq(r2_sub(x12, q)) = rad and
            r2_norm_sq(r2_sub(x23, q)) = rad and
            r2_norm_sq(r2_sub(x31, q)) = rad
        exists(c: Pair[Real, Real]) {
            r2_norm_sq(r2_sub(x12, q)) = rad and
            r2_norm_sq(r2_sub(x23, q)) = rad and
            r2_norm_sq(r2_sub(x31, q)) = rad
        }
    }
}
