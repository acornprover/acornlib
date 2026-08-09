from pair import Pair
from real import Real, mul_left_cancel, div_mul_cancel_left
from top100 import r2_add, r2_neg, r2_sub, r2_dot, r2_norm_sq
from theorems1000.r2_helpers import r2_smul, r2_norm_sq_smul, r2_rot90, r2_rot90_norm_sq,
    r2_rot90_add, r2_rot90_sub, r2_rot90_smul, r2_rot90_neg, r2_rot90_twice,
    r2_dot_rot90_self, r2_dot_smul_right, r2_norm_sq_add_expansion, r2_norm_sq_neg,
    r2_neg_add, r2_neg_neg, r2_smul_double, r2_smul_smul, r2_smul_add_distrib,
    r2_smul_neg, r2_smul_sub_distrib, r2_smul_add_self, r2_smul_sub_self,
    r2_sub_eq_add_neg, r2_add_neg_right, r2_add_zero_left, r2_add_comm, r2_add_assoc,
    r2_sub_add_pair_distrib, r2_sub_reverse_neg, r2_sub_through, r2_sub_sub_sum_rearrange,
    r2_sub_sub_sub_pair, r2_add_pair_rearrange, r2_add_zero_right, r2_double_eq_double,
    real_double_eq_double, real_double_eq_add_self, real_two_neq_zero, r2_norm_sq_double

// Napoleon's theorem: with equilateral triangles erected on the sides of any
// triangle, the centers of the three equilateral triangles themselves form an
// equilateral triangle.
//
// A rotation by 60 degrees is represented with a parameter `s` with
// `s * s = 3` (a formal square root of three): `2 R60(v) = v - s J v` where
// `J` is the quarter turn.  The apex of the equilateral triangle on the side
// `ab` is `a + R60(b - a)`; its center `n1` satisfies `3 n1 = a + b + apex`,
// hence, clearing the halves, `6 n1 = 3 (a + b) - s J(b - a)`.  Writing
// `u = a - b` and `w = b - c` for two side displacements, the doubled
// difference of consecutive centers is
//     V1 = 6 (n1 - n2) = 3 (u + w) + s J(u - w),
//     V2 = 6 (n2 - n3) = 3 (w + (c - a)) + s J(w - (c - a)),
//     V3 = 6 (n3 - n1) = 3 ((c - a) + u) + s J((c - a) - u),
// and since `c - a = -(u + w)` the three displacements are the cyclic
// rotations of one another: with `D120(x) = -(x + s J x)` the doubled
// rotation by 120 degrees, `D120(V1) = 2 V3` and `D120(V3) = 2 V2`.  As
// `|D120(x)|^2 = 4 |x|^2` (from `s^2 = 3`, `|J x|^2 = |x|^2` and
// `x . J x = 0`), the three squared norms agree and the triangle `n1 n2 n3`
// is equilateral.  The computation is carried out below.

// ---------------------------------------------------------------------------
// The doubled difference of the centers of two adjacent equilateral
// triangles.
// ---------------------------------------------------------------------------

/// The doubled difference of the centers of the equilateral triangles on the
/// sides `ab` and `bc`:
/// `6 (n1 - n2) = 3 (u + w) + s J(u - w)` for `u = a - b`, `w = b - c`.
theorem napoleon_six_sub(
    a: Pair[Real, Real], b: Pair[Real, Real], c: Pair[Real, Real], s: Real,
    n1: Pair[Real, Real], n2: Pair[Real, Real],
    u: Pair[Real, Real], w: Pair[Real, Real]
) {
    r2_smul((Real.1 + Real.1) * (Real.1 + Real.1 + Real.1), n1) =
        r2_sub(r2_smul((Real.1 + Real.1 + Real.1), r2_add(a, b)), r2_smul(s, r2_rot90(r2_sub(b, a)))) and
    r2_smul((Real.1 + Real.1) * (Real.1 + Real.1 + Real.1), n2) =
        r2_sub(r2_smul((Real.1 + Real.1 + Real.1), r2_add(b, c)), r2_smul(s, r2_rot90(r2_sub(c, b)))) and
    u = r2_sub(a, b) and
    w = r2_sub(b, c)
    implies
    r2_smul((Real.1 + Real.1) * (Real.1 + Real.1 + Real.1), r2_sub(n1, n2)) =
        r2_add(
            r2_smul((Real.1 + Real.1 + Real.1), r2_add(u, w)),
            r2_smul(s, r2_rot90(r2_sub(u, w))))
} by {
    if r2_smul((Real.1 + Real.1) * (Real.1 + Real.1 + Real.1), n1) =
        r2_sub(r2_smul((Real.1 + Real.1 + Real.1), r2_add(a, b)), r2_smul(s, r2_rot90(r2_sub(b, a)))) and
        r2_smul((Real.1 + Real.1) * (Real.1 + Real.1 + Real.1), n2) =
        r2_sub(r2_smul((Real.1 + Real.1 + Real.1), r2_add(b, c)), r2_smul(s, r2_rot90(r2_sub(c, b)))) and
        u = r2_sub(a, b) and
        w = r2_sub(b, c) {
        r2_smul((Real.1 + Real.1) * (Real.1 + Real.1 + Real.1), n1) =
            r2_sub(r2_smul((Real.1 + Real.1 + Real.1), r2_add(a, b)), r2_smul(s, r2_rot90(r2_sub(b, a))))
        r2_smul((Real.1 + Real.1) * (Real.1 + Real.1 + Real.1), n2) =
            r2_sub(r2_smul((Real.1 + Real.1 + Real.1), r2_add(b, c)), r2_smul(s, r2_rot90(r2_sub(c, b))))
        u = r2_sub(a, b)
        w = r2_sub(b, c)
        r2_smul_sub_distrib((Real.1 + Real.1) * (Real.1 + Real.1 + Real.1), n1, n2)
        r2_smul((Real.1 + Real.1) * (Real.1 + Real.1 + Real.1), r2_sub(n1, n2)) =
            r2_sub(
                r2_smul((Real.1 + Real.1) * (Real.1 + Real.1 + Real.1), n1),
                r2_smul((Real.1 + Real.1) * (Real.1 + Real.1 + Real.1), n2))
        r2_smul((Real.1 + Real.1) * (Real.1 + Real.1 + Real.1), r2_sub(n1, n2)) =
            r2_sub(
                r2_sub(
                    r2_smul((Real.1 + Real.1 + Real.1), r2_add(a, b)),
                    r2_smul(s, r2_rot90(r2_sub(b, a)))),
                r2_sub(
                    r2_smul((Real.1 + Real.1 + Real.1), r2_add(b, c)),
                    r2_smul(s, r2_rot90(r2_sub(c, b)))))
        r2_sub_sub_sub_pair(
            r2_smul((Real.1 + Real.1 + Real.1), r2_add(a, b)),
            r2_smul(s, r2_rot90(r2_sub(b, a))),
            r2_smul((Real.1 + Real.1 + Real.1), r2_add(b, c)),
            r2_smul(s, r2_rot90(r2_sub(c, b))))
        r2_smul((Real.1 + Real.1) * (Real.1 + Real.1 + Real.1), r2_sub(n1, n2)) =
            r2_add(
                r2_sub(
                    r2_smul((Real.1 + Real.1 + Real.1), r2_add(a, b)),
                    r2_smul((Real.1 + Real.1 + Real.1), r2_add(b, c))),
                r2_sub(
                    r2_smul(s, r2_rot90(r2_sub(c, b))),
                    r2_smul(s, r2_rot90(r2_sub(b, a)))))
        r2_smul_sub_distrib((Real.1 + Real.1 + Real.1), r2_add(a, b), r2_add(b, c))
        r2_smul((Real.1 + Real.1 + Real.1), r2_sub(r2_add(a, b), r2_add(b, c))) =
            r2_sub(
                r2_smul((Real.1 + Real.1 + Real.1), r2_add(a, b)),
                r2_smul((Real.1 + Real.1 + Real.1), r2_add(b, c)))
        r2_sub(
            r2_smul((Real.1 + Real.1 + Real.1), r2_add(a, b)),
            r2_smul((Real.1 + Real.1 + Real.1), r2_add(b, c))) =
            r2_smul((Real.1 + Real.1 + Real.1), r2_sub(r2_add(a, b), r2_add(b, c)))
        r2_sub_add_pair_distrib(a, b, b, c)
        r2_sub(r2_add(a, b), r2_add(b, c)) = r2_add(r2_sub(a, b), r2_sub(b, c))
        r2_sub(r2_add(a, b), r2_add(b, c)) = r2_add(u, w)
        r2_sub(
            r2_smul((Real.1 + Real.1 + Real.1), r2_add(a, b)),
            r2_smul((Real.1 + Real.1 + Real.1), r2_add(b, c))) =
            r2_smul((Real.1 + Real.1 + Real.1), r2_add(u, w))
        r2_smul_sub_distrib(s, r2_rot90(r2_sub(c, b)), r2_rot90(r2_sub(b, a)))
        r2_smul(s, r2_sub(r2_rot90(r2_sub(c, b)), r2_rot90(r2_sub(b, a)))) =
            r2_sub(
                r2_smul(s, r2_rot90(r2_sub(c, b))),
                r2_smul(s, r2_rot90(r2_sub(b, a))))
        r2_sub(
            r2_smul(s, r2_rot90(r2_sub(c, b))),
            r2_smul(s, r2_rot90(r2_sub(b, a)))) =
            r2_smul(s, r2_sub(r2_rot90(r2_sub(c, b)), r2_rot90(r2_sub(b, a))))
        r2_rot90_sub(r2_sub(c, b), r2_sub(b, a))
        r2_rot90(r2_sub(r2_sub(c, b), r2_sub(b, a))) =
            r2_sub(r2_rot90(r2_sub(c, b)), r2_rot90(r2_sub(b, a)))
        r2_sub(r2_rot90(r2_sub(c, b)), r2_rot90(r2_sub(b, a))) =
            r2_rot90(r2_sub(r2_sub(c, b), r2_sub(b, a)))
        r2_sub_sub_sum_rearrange(b, b, c, a)
        r2_sub(r2_sub(c, b), r2_sub(b, a)) = r2_sub(r2_add(c, a), r2_add(b, b))
        r2_add_comm(c, a)
        r2_add(c, a) = r2_add(a, c)
        r2_sub(r2_sub(c, b), r2_sub(b, a)) = r2_sub(r2_add(a, c), r2_add(b, b))
        r2_sub_sub_sum_rearrange(b, b, a, c)
        r2_sub(r2_sub(a, b), r2_sub(b, c)) = r2_sub(r2_add(a, c), r2_add(b, b))
        r2_sub(r2_sub(a, b), r2_sub(b, c)) = r2_sub(u, w)
        r2_sub(r2_sub(c, b), r2_sub(b, a)) = r2_sub(u, w)
        r2_sub(r2_rot90(r2_sub(c, b)), r2_rot90(r2_sub(b, a))) = r2_rot90(r2_sub(u, w))
        r2_sub(
            r2_smul(s, r2_rot90(r2_sub(c, b))),
            r2_smul(s, r2_rot90(r2_sub(b, a)))) =
            r2_smul(s, r2_rot90(r2_sub(u, w)))
        r2_smul((Real.1 + Real.1) * (Real.1 + Real.1 + Real.1), r2_sub(n1, n2)) =
            r2_add(
                r2_smul((Real.1 + Real.1 + Real.1), r2_add(u, w)),
                r2_smul(s, r2_rot90(r2_sub(u, w))))
    }
}

// ---------------------------------------------------------------------------
// The rotation by 120 degrees: D120(x) = -(x + s J x) has norm-squared
// four times the norm-squared of x, and maps the doubled center differences
// onto each other cyclically.
// ---------------------------------------------------------------------------

/// The squared norm of the doubled 120-degree rotation:
/// `|D120(x)|^2 = 4 |x|^2` when `s * s = 3`.
theorem napoleon_d120_norm(s: Real, x: Pair[Real, Real]) {
    s * s = Real.1 + Real.1 + Real.1 implies
    r2_norm_sq(r2_neg(r2_add(x, r2_smul(s, r2_rot90(x))))) =
        (Real.1 + Real.1 + Real.1 + Real.1) * r2_norm_sq(x)
} by {
    if s * s = Real.1 + Real.1 + Real.1 {
        s * s = Real.1 + Real.1 + Real.1
        r2_norm_sq_neg(r2_add(x, r2_smul(s, r2_rot90(x))))
        r2_norm_sq(r2_neg(r2_add(x, r2_smul(s, r2_rot90(x))))) =
            r2_norm_sq(r2_add(x, r2_smul(s, r2_rot90(x))))
        r2_norm_sq_add_expansion(x, r2_smul(s, r2_rot90(x)))
        r2_norm_sq(r2_add(x, r2_smul(s, r2_rot90(x)))) =
            r2_norm_sq(x) + r2_norm_sq(r2_smul(s, r2_rot90(x))) +
            r2_dot(x, r2_smul(s, r2_rot90(x))) + r2_dot(x, r2_smul(s, r2_rot90(x)))
        r2_norm_sq_smul(s, r2_rot90(x))
        r2_norm_sq(r2_smul(s, r2_rot90(x))) = s * s * r2_norm_sq(r2_rot90(x))
        r2_rot90_norm_sq(x)
        r2_norm_sq(r2_rot90(x)) = r2_norm_sq(x)
        r2_norm_sq(r2_smul(s, r2_rot90(x))) = s * s * r2_norm_sq(x)
        r2_norm_sq(r2_smul(s, r2_rot90(x))) = (Real.1 + Real.1 + Real.1) * r2_norm_sq(x)
        r2_dot_smul_right(s, x, r2_rot90(x))
        r2_dot(x, r2_smul(s, r2_rot90(x))) = s * r2_dot(x, r2_rot90(x))
        r2_dot_rot90_self(x)
        r2_dot(x, r2_rot90(x)) = Real.0
        s * r2_dot(x, r2_rot90(x)) = Real.0
        r2_dot(x, r2_smul(s, r2_rot90(x))) = Real.0
        r2_norm_sq(r2_add(x, r2_smul(s, r2_rot90(x)))) =
            r2_norm_sq(x) + (Real.1 + Real.1 + Real.1) * r2_norm_sq(x) + Real.0 + Real.0
        r2_norm_sq(x) + (Real.1 + Real.1 + Real.1) * r2_norm_sq(x) + Real.0 + Real.0 =
            (Real.1 + Real.1 + Real.1 + Real.1) * r2_norm_sq(x)
        r2_norm_sq(r2_neg(r2_add(x, r2_smul(s, r2_rot90(x))))) =
            (Real.1 + Real.1 + Real.1 + Real.1) * r2_norm_sq(x)
    }
}

/// `(u + w) - (u - w) = w + w`.
theorem napoleon_sum_sub(u: Pair[Real, Real], w: Pair[Real, Real]) {
    r2_sub(r2_add(u, w), r2_sub(u, w)) = r2_add(w, w)
} by {
    r2_sub_eq_add_neg(u, w)
    r2_sub(u, w) = r2_add(u, r2_neg(w))
    r2_sub_add_pair_distrib(u, w, u, r2_neg(w))
    r2_sub(r2_add(u, w), r2_add(u, r2_neg(w))) = r2_add(r2_sub(u, u), r2_sub(w, r2_neg(w)))
    r2_sub(r2_add(u, w), r2_sub(u, w)) = r2_add(r2_sub(u, u), r2_sub(w, r2_neg(w)))
    r2_sub(w, r2_neg(w)) = r2_add(w, w)
    r2_sub(u, u) = Pair.new(Real.0, Real.0)
    r2_add_zero_left(r2_add(w, w))
    r2_add(r2_sub(u, u), r2_sub(w, r2_neg(w))) = r2_add(w, w)
    r2_sub(r2_add(u, w), r2_sub(u, w)) = r2_add(w, w)
}

/// `3(u + w) - 3(u - w) = 6 w`.
theorem napoleon_three_sub(u: Pair[Real, Real], w: Pair[Real, Real]) {
    r2_sub(
        r2_smul((Real.1 + Real.1 + Real.1), r2_add(u, w)),
        r2_smul((Real.1 + Real.1 + Real.1), r2_sub(u, w))) =
        r2_smul((Real.1 + Real.1) * (Real.1 + Real.1 + Real.1), w)
} by {
    napoleon_sum_sub(u, w)
    r2_sub(r2_add(u, w), r2_sub(u, w)) = r2_add(w, w)
    r2_smul_sub_distrib((Real.1 + Real.1 + Real.1), r2_add(u, w), r2_sub(u, w))
    r2_smul((Real.1 + Real.1 + Real.1), r2_sub(r2_add(u, w), r2_sub(u, w))) =
        r2_sub(
            r2_smul((Real.1 + Real.1 + Real.1), r2_add(u, w)),
            r2_smul((Real.1 + Real.1 + Real.1), r2_sub(u, w)))
    r2_smul((Real.1 + Real.1 + Real.1), r2_sub(r2_add(u, w), r2_sub(u, w))) =
        r2_smul((Real.1 + Real.1 + Real.1), r2_add(w, w))
    r2_smul_add_distrib((Real.1 + Real.1 + Real.1), w, w)
    r2_smul((Real.1 + Real.1 + Real.1), r2_add(w, w)) =
        r2_add(r2_smul((Real.1 + Real.1 + Real.1), w), r2_smul((Real.1 + Real.1 + Real.1), w))
    r2_smul_double(r2_smul((Real.1 + Real.1 + Real.1), w))
    r2_smul((Real.1 + Real.1), r2_smul((Real.1 + Real.1 + Real.1), w)) =
        r2_add(r2_smul((Real.1 + Real.1 + Real.1), w), r2_smul((Real.1 + Real.1 + Real.1), w))
    r2_smul_smul((Real.1 + Real.1), (Real.1 + Real.1 + Real.1), w)
    r2_smul((Real.1 + Real.1), r2_smul((Real.1 + Real.1 + Real.1), w)) =
        r2_smul((Real.1 + Real.1) * (Real.1 + Real.1 + Real.1), w)
    r2_smul((Real.1 + Real.1 + Real.1), r2_add(w, w)) =
        r2_smul((Real.1 + Real.1) * (Real.1 + Real.1 + Real.1), w)
    r2_sub(
        r2_smul((Real.1 + Real.1 + Real.1), r2_add(u, w)),
        r2_smul((Real.1 + Real.1 + Real.1), r2_sub(u, w))) =
        r2_smul((Real.1 + Real.1) * (Real.1 + Real.1 + Real.1), w)
}

/// `(u - w) + 3(u + w) = 4 u + 2 w`.
theorem napoleon_sum_three(u: Pair[Real, Real], w: Pair[Real, Real]) {
    r2_add(r2_sub(u, w), r2_smul((Real.1 + Real.1 + Real.1), r2_add(u, w))) =
        r2_add(r2_smul((Real.1 + Real.1 + Real.1 + Real.1), u), r2_smul((Real.1 + Real.1), w))
} by {
    r2_smul_add_distrib((Real.1 + Real.1 + Real.1), u, w)
    r2_smul((Real.1 + Real.1 + Real.1), r2_add(u, w)) =
        r2_add(r2_smul((Real.1 + Real.1 + Real.1), u), r2_smul((Real.1 + Real.1 + Real.1), w))
    r2_add(r2_sub(u, w), r2_smul((Real.1 + Real.1 + Real.1), r2_add(u, w))) =
        r2_add(
            r2_sub(u, w),
            r2_add(r2_smul((Real.1 + Real.1 + Real.1), u), r2_smul((Real.1 + Real.1 + Real.1), w)))
    r2_sub_eq_add_neg(u, w)
    r2_sub(u, w) = r2_add(u, r2_neg(w))
    r2_add(r2_sub(u, w), r2_add(r2_smul((Real.1 + Real.1 + Real.1), u), r2_smul((Real.1 + Real.1 + Real.1), w))) =
        r2_add(
            r2_add(u, r2_neg(w)),
            r2_add(r2_smul((Real.1 + Real.1 + Real.1), u), r2_smul((Real.1 + Real.1 + Real.1), w)))
    r2_smul_add_self((Real.1 + Real.1 + Real.1), u)
    r2_add(r2_smul((Real.1 + Real.1 + Real.1), u), u) = r2_smul((Real.1 + Real.1 + Real.1 + Real.1), u)
    r2_add_comm(r2_smul((Real.1 + Real.1 + Real.1), u), u)
    r2_add(r2_smul((Real.1 + Real.1 + Real.1), u), u) = r2_add(u, r2_smul((Real.1 + Real.1 + Real.1), u))
    r2_add(u, r2_smul((Real.1 + Real.1 + Real.1), u)) = r2_smul((Real.1 + Real.1 + Real.1 + Real.1), u)
    r2_add_pair_rearrange(
        u, r2_neg(w), r2_smul((Real.1 + Real.1 + Real.1), u), r2_smul((Real.1 + Real.1 + Real.1), w))
    r2_add(
        r2_add(u, r2_neg(w)),
        r2_add(r2_smul((Real.1 + Real.1 + Real.1), u), r2_smul((Real.1 + Real.1 + Real.1), w))) =
        r2_add(
            r2_add(u, r2_smul((Real.1 + Real.1 + Real.1), u)),
            r2_add(r2_neg(w), r2_smul((Real.1 + Real.1 + Real.1), w)))
    r2_add_comm(r2_smul((Real.1 + Real.1 + Real.1), w), r2_neg(w))
    r2_add(r2_neg(w), r2_smul((Real.1 + Real.1 + Real.1), w)) =
        r2_add(r2_smul((Real.1 + Real.1 + Real.1), w), r2_neg(w))
    r2_sub_eq_add_neg(r2_smul((Real.1 + Real.1 + Real.1), w), w)
    r2_sub(r2_smul((Real.1 + Real.1 + Real.1), w), w) =
        r2_add(r2_smul((Real.1 + Real.1 + Real.1), w), r2_neg(w))
    r2_add(r2_neg(w), r2_smul((Real.1 + Real.1 + Real.1), w)) =
        r2_sub(r2_smul((Real.1 + Real.1 + Real.1), w), w)
    r2_smul_sub_self((Real.1 + Real.1 + Real.1), w)
    r2_sub(r2_smul((Real.1 + Real.1 + Real.1), w), w) = r2_smul((Real.1 + Real.1), w)
    r2_add(r2_neg(w), r2_smul((Real.1 + Real.1 + Real.1), w)) = r2_smul((Real.1 + Real.1), w)
    r2_add(
        r2_add(u, r2_neg(w)),
        r2_add(r2_smul((Real.1 + Real.1 + Real.1), u), r2_smul((Real.1 + Real.1 + Real.1), w))) =
        r2_add(r2_smul((Real.1 + Real.1 + Real.1 + Real.1), u), r2_smul((Real.1 + Real.1), w))
    r2_add(r2_sub(u, w), r2_smul((Real.1 + Real.1 + Real.1), r2_add(u, w))) =
        r2_add(r2_smul((Real.1 + Real.1 + Real.1 + Real.1), u), r2_smul((Real.1 + Real.1), w))
}

/// The common closed form of the three doubled center differences:
/// `C(u, w) = -(6 w + 4 s J u + 2 s J w)`.
define napoleon_c(s: Real, u: Pair[Real, Real], w: Pair[Real, Real]) -> Pair[Real, Real] {
    r2_neg(r2_add(
        r2_smul((Real.1 + Real.1) * (Real.1 + Real.1 + Real.1), w),
        r2_add(
            r2_smul((Real.1 + Real.1 + Real.1 + Real.1), r2_smul(s, r2_rot90(u))),
            r2_smul((Real.1 + Real.1), r2_smul(s, r2_rot90(w))))))
}

/// The doubled center difference `V1 = 3(u + w) + s J(u - w)` satisfies
/// `D120(V1) = C(u, w)`.
theorem napoleon_rot_form1(s: Real, u: Pair[Real, Real], w: Pair[Real, Real]) {
    s * s = Real.1 + Real.1 + Real.1 implies
    r2_neg(r2_add(
        r2_add(r2_smul((Real.1 + Real.1 + Real.1), r2_add(u, w)), r2_smul(s, r2_rot90(r2_sub(u, w)))),
        r2_smul(s, r2_rot90(r2_add(
            r2_smul((Real.1 + Real.1 + Real.1), r2_add(u, w)),
            r2_smul(s, r2_rot90(r2_sub(u, w)))))))) =
        napoleon_c(s, u, w)
} by {
    if s * s = Real.1 + Real.1 + Real.1 {
        s * s = Real.1 + Real.1 + Real.1
        // J(3(u + w) + s J(u - w)) = 3 J(u + w) - s (u - w)
        r2_rot90_add(
            r2_smul((Real.1 + Real.1 + Real.1), r2_add(u, w)),
            r2_smul(s, r2_rot90(r2_sub(u, w))))
        r2_rot90(r2_add(
            r2_smul((Real.1 + Real.1 + Real.1), r2_add(u, w)),
            r2_smul(s, r2_rot90(r2_sub(u, w))))) =
            r2_add(
                r2_rot90(r2_smul((Real.1 + Real.1 + Real.1), r2_add(u, w))),
                r2_rot90(r2_smul(s, r2_rot90(r2_sub(u, w)))))
        r2_rot90_smul((Real.1 + Real.1 + Real.1), r2_add(u, w))
        r2_rot90(r2_smul((Real.1 + Real.1 + Real.1), r2_add(u, w))) =
            r2_smul((Real.1 + Real.1 + Real.1), r2_rot90(r2_add(u, w)))
        r2_rot90_smul(s, r2_rot90(r2_sub(u, w)))
        r2_rot90(r2_smul(s, r2_rot90(r2_sub(u, w)))) =
            r2_smul(s, r2_rot90(r2_rot90(r2_sub(u, w))))
        r2_rot90_twice(r2_sub(u, w))
        r2_rot90(r2_rot90(r2_sub(u, w))) = r2_neg(r2_sub(u, w))
        r2_smul(s, r2_rot90(r2_rot90(r2_sub(u, w)))) = r2_smul(s, r2_neg(r2_sub(u, w)))
        r2_smul_neg(s, r2_sub(u, w))
        r2_smul(s, r2_neg(r2_sub(u, w))) = r2_neg(r2_smul(s, r2_sub(u, w)))
        r2_rot90(r2_smul(s, r2_rot90(r2_sub(u, w)))) = r2_neg(r2_smul(s, r2_sub(u, w)))
        r2_rot90(r2_add(
            r2_smul((Real.1 + Real.1 + Real.1), r2_add(u, w)),
            r2_smul(s, r2_rot90(r2_sub(u, w))))) =
            r2_add(
                r2_smul((Real.1 + Real.1 + Real.1), r2_rot90(r2_add(u, w))),
                r2_neg(r2_smul(s, r2_sub(u, w))))
        // s J(F) = 3s J(u + w) - 3 (u - w)
        r2_smul_add_distrib(
            s,
            r2_smul((Real.1 + Real.1 + Real.1), r2_rot90(r2_add(u, w))),
            r2_neg(r2_smul(s, r2_sub(u, w))))
        r2_smul(s, r2_add(
            r2_smul((Real.1 + Real.1 + Real.1), r2_rot90(r2_add(u, w))),
            r2_neg(r2_smul(s, r2_sub(u, w))))) =
            r2_add(
                r2_smul(s, r2_smul((Real.1 + Real.1 + Real.1), r2_rot90(r2_add(u, w)))),
                r2_smul(s, r2_neg(r2_smul(s, r2_sub(u, w)))))
        r2_smul_smul(s, (Real.1 + Real.1 + Real.1), r2_rot90(r2_add(u, w)))
        r2_smul(s, r2_smul((Real.1 + Real.1 + Real.1), r2_rot90(r2_add(u, w)))) =
            r2_smul(s * (Real.1 + Real.1 + Real.1), r2_rot90(r2_add(u, w)))
        r2_smul_neg(s, r2_smul(s, r2_sub(u, w)))
        r2_smul(s, r2_neg(r2_smul(s, r2_sub(u, w)))) = r2_neg(r2_smul(s, r2_smul(s, r2_sub(u, w))))
        r2_smul_smul(s, s, r2_sub(u, w))
        r2_smul(s, r2_smul(s, r2_sub(u, w))) = r2_smul(s * s, r2_sub(u, w))
        r2_smul(s, r2_smul(s, r2_sub(u, w))) = r2_smul((Real.1 + Real.1 + Real.1), r2_sub(u, w))
        r2_smul(s, r2_neg(r2_smul(s, r2_sub(u, w)))) = r2_neg(r2_smul((Real.1 + Real.1 + Real.1), r2_sub(u, w)))
        r2_smul(s, r2_rot90(r2_add(
            r2_smul((Real.1 + Real.1 + Real.1), r2_add(u, w)),
            r2_smul(s, r2_rot90(r2_sub(u, w)))))) =
            r2_add(
                r2_smul(s * (Real.1 + Real.1 + Real.1), r2_rot90(r2_add(u, w))),
                r2_neg(r2_smul((Real.1 + Real.1 + Real.1), r2_sub(u, w))))
        // F + s J(F) regrouped as (3(u+w) - 3(u-w)) + (s J(u-w) + 3s J(u+w))
        r2_add_comm(
            r2_smul(s * (Real.1 + Real.1 + Real.1), r2_rot90(r2_add(u, w))),
            r2_neg(r2_smul((Real.1 + Real.1 + Real.1), r2_sub(u, w))))
        r2_add(
            r2_smul(s * (Real.1 + Real.1 + Real.1), r2_rot90(r2_add(u, w))),
            r2_neg(r2_smul((Real.1 + Real.1 + Real.1), r2_sub(u, w)))) =
            r2_add(
                r2_neg(r2_smul((Real.1 + Real.1 + Real.1), r2_sub(u, w))),
                r2_smul(s * (Real.1 + Real.1 + Real.1), r2_rot90(r2_add(u, w))))
        r2_add_pair_rearrange(
            r2_smul((Real.1 + Real.1 + Real.1), r2_add(u, w)),
            r2_smul(s, r2_rot90(r2_sub(u, w))),
            r2_neg(r2_smul((Real.1 + Real.1 + Real.1), r2_sub(u, w))),
            r2_smul(s * (Real.1 + Real.1 + Real.1), r2_rot90(r2_add(u, w))))
        r2_add(
            r2_add(r2_smul((Real.1 + Real.1 + Real.1), r2_add(u, w)), r2_smul(s, r2_rot90(r2_sub(u, w)))),
            r2_add(
                r2_smul(s * (Real.1 + Real.1 + Real.1), r2_rot90(r2_add(u, w))),
                r2_neg(r2_smul((Real.1 + Real.1 + Real.1), r2_sub(u, w))))) =
            r2_add(
                r2_add(
                    r2_smul((Real.1 + Real.1 + Real.1), r2_add(u, w)),
                    r2_neg(r2_smul((Real.1 + Real.1 + Real.1), r2_sub(u, w)))),
                r2_add(
                    r2_smul(s, r2_rot90(r2_sub(u, w))),
                    r2_smul(s * (Real.1 + Real.1 + Real.1), r2_rot90(r2_add(u, w)))))
        r2_sub_eq_add_neg(
            r2_smul((Real.1 + Real.1 + Real.1), r2_add(u, w)),
            r2_smul((Real.1 + Real.1 + Real.1), r2_sub(u, w)))
        r2_sub(
            r2_smul((Real.1 + Real.1 + Real.1), r2_add(u, w)),
            r2_smul((Real.1 + Real.1 + Real.1), r2_sub(u, w))) =
            r2_add(
                r2_smul((Real.1 + Real.1 + Real.1), r2_add(u, w)),
                r2_neg(r2_smul((Real.1 + Real.1 + Real.1), r2_sub(u, w))))
        r2_add(
            r2_add(r2_smul((Real.1 + Real.1 + Real.1), r2_add(u, w)), r2_smul(s, r2_rot90(r2_sub(u, w)))),
            r2_add(
                r2_smul(s * (Real.1 + Real.1 + Real.1), r2_rot90(r2_add(u, w))),
                r2_neg(r2_smul((Real.1 + Real.1 + Real.1), r2_sub(u, w))))) =
            r2_add(
                r2_sub(
                    r2_smul((Real.1 + Real.1 + Real.1), r2_add(u, w)),
                    r2_smul((Real.1 + Real.1 + Real.1), r2_sub(u, w))),
                r2_add(
                    r2_smul(s, r2_rot90(r2_sub(u, w))),
                    r2_smul(s * (Real.1 + Real.1 + Real.1), r2_rot90(r2_add(u, w)))))
        // 3(u + w) - 3(u - w) = 6 w
        napoleon_three_sub(u, w)
        r2_sub(
            r2_smul((Real.1 + Real.1 + Real.1), r2_add(u, w)),
            r2_smul((Real.1 + Real.1 + Real.1), r2_sub(u, w))) =
            r2_smul((Real.1 + Real.1) * (Real.1 + Real.1 + Real.1), w)
        // s J(u - w) + 3s J(u + w) = 4 s J u + 2 s J w
        r2_smul_smul(s, (Real.1 + Real.1 + Real.1), r2_rot90(r2_add(u, w)))
        r2_smul(s, r2_smul((Real.1 + Real.1 + Real.1), r2_rot90(r2_add(u, w)))) =
            r2_smul(s * (Real.1 + Real.1 + Real.1), r2_rot90(r2_add(u, w)))
        r2_rot90_smul((Real.1 + Real.1 + Real.1), r2_add(u, w))
        r2_smul((Real.1 + Real.1 + Real.1), r2_rot90(r2_add(u, w))) =
            r2_rot90(r2_smul((Real.1 + Real.1 + Real.1), r2_add(u, w)))
        r2_add(
            r2_smul(s, r2_rot90(r2_sub(u, w))),
            r2_smul(s * (Real.1 + Real.1 + Real.1), r2_rot90(r2_add(u, w)))) =
            r2_add(
                r2_smul(s, r2_rot90(r2_sub(u, w))),
                r2_smul(s, r2_smul((Real.1 + Real.1 + Real.1), r2_rot90(r2_add(u, w)))))
        r2_smul_add_distrib(
            s, r2_rot90(r2_sub(u, w)), r2_smul((Real.1 + Real.1 + Real.1), r2_rot90(r2_add(u, w))))
        r2_smul(s, r2_add(
            r2_rot90(r2_sub(u, w)),
            r2_smul((Real.1 + Real.1 + Real.1), r2_rot90(r2_add(u, w))))) =
            r2_add(
                r2_smul(s, r2_rot90(r2_sub(u, w))),
                r2_smul(s, r2_smul((Real.1 + Real.1 + Real.1), r2_rot90(r2_add(u, w)))))
        r2_add(
            r2_smul(s, r2_rot90(r2_sub(u, w))),
            r2_smul(s * (Real.1 + Real.1 + Real.1), r2_rot90(r2_add(u, w)))) =
            r2_smul(s, r2_add(
                r2_rot90(r2_sub(u, w)),
                r2_smul((Real.1 + Real.1 + Real.1), r2_rot90(r2_add(u, w)))))
        r2_add(
            r2_rot90(r2_sub(u, w)),
            r2_smul((Real.1 + Real.1 + Real.1), r2_rot90(r2_add(u, w)))) =
            r2_add(
                r2_rot90(r2_sub(u, w)),
                r2_rot90(r2_smul((Real.1 + Real.1 + Real.1), r2_add(u, w))))
        r2_rot90_add(r2_sub(u, w), r2_smul((Real.1 + Real.1 + Real.1), r2_add(u, w)))
        r2_rot90(r2_add(
            r2_sub(u, w),
            r2_smul((Real.1 + Real.1 + Real.1), r2_add(u, w)))) =
            r2_add(
                r2_rot90(r2_sub(u, w)),
                r2_rot90(r2_smul((Real.1 + Real.1 + Real.1), r2_add(u, w))))
        r2_add(
            r2_rot90(r2_sub(u, w)),
            r2_smul((Real.1 + Real.1 + Real.1), r2_rot90(r2_add(u, w)))) =
            r2_rot90(r2_add(
                r2_sub(u, w),
                r2_smul((Real.1 + Real.1 + Real.1), r2_add(u, w))))
        napoleon_sum_three(u, w)
        r2_add(r2_sub(u, w), r2_smul((Real.1 + Real.1 + Real.1), r2_add(u, w))) =
            r2_add(r2_smul((Real.1 + Real.1 + Real.1 + Real.1), u), r2_smul((Real.1 + Real.1), w))
        r2_add(
            r2_rot90(r2_sub(u, w)),
            r2_smul((Real.1 + Real.1 + Real.1), r2_rot90(r2_add(u, w)))) =
            r2_rot90(r2_add(
                r2_smul((Real.1 + Real.1 + Real.1 + Real.1), u), r2_smul((Real.1 + Real.1), w)))
        r2_rot90_add(
            r2_smul((Real.1 + Real.1 + Real.1 + Real.1), u), r2_smul((Real.1 + Real.1), w))
        r2_rot90(r2_add(
            r2_smul((Real.1 + Real.1 + Real.1 + Real.1), u), r2_smul((Real.1 + Real.1), w))) =
            r2_add(
                r2_rot90(r2_smul((Real.1 + Real.1 + Real.1 + Real.1), u)),
                r2_rot90(r2_smul((Real.1 + Real.1), w)))
        r2_rot90_smul((Real.1 + Real.1 + Real.1 + Real.1), u)
        r2_rot90(r2_smul((Real.1 + Real.1 + Real.1 + Real.1), u)) =
            r2_smul((Real.1 + Real.1 + Real.1 + Real.1), r2_rot90(u))
        r2_rot90_smul((Real.1 + Real.1), w)
        r2_rot90(r2_smul((Real.1 + Real.1), w)) = r2_smul((Real.1 + Real.1), r2_rot90(w))
        r2_add(
            r2_rot90(r2_sub(u, w)),
            r2_smul((Real.1 + Real.1 + Real.1), r2_rot90(r2_add(u, w)))) =
            r2_add(
                r2_smul((Real.1 + Real.1 + Real.1 + Real.1), r2_rot90(u)),
                r2_smul((Real.1 + Real.1), r2_rot90(w)))
        r2_add(
            r2_smul(s, r2_rot90(r2_sub(u, w))),
            r2_smul(s * (Real.1 + Real.1 + Real.1), r2_rot90(r2_add(u, w)))) =
            r2_smul(s, r2_add(
                r2_smul((Real.1 + Real.1 + Real.1 + Real.1), r2_rot90(u)),
                r2_smul((Real.1 + Real.1), r2_rot90(w))))
        r2_smul_add_distrib(
            s,
            r2_smul((Real.1 + Real.1 + Real.1 + Real.1), r2_rot90(u)),
            r2_smul((Real.1 + Real.1), r2_rot90(w)))
        r2_smul(s, r2_add(
            r2_smul((Real.1 + Real.1 + Real.1 + Real.1), r2_rot90(u)),
            r2_smul((Real.1 + Real.1), r2_rot90(w)))) =
            r2_add(
                r2_smul(s, r2_smul((Real.1 + Real.1 + Real.1 + Real.1), r2_rot90(u))),
                r2_smul(s, r2_smul((Real.1 + Real.1), r2_rot90(w))))
        r2_smul_smul(s, (Real.1 + Real.1 + Real.1 + Real.1), r2_rot90(u))
        r2_smul(s, r2_smul((Real.1 + Real.1 + Real.1 + Real.1), r2_rot90(u))) =
            r2_smul(s * (Real.1 + Real.1 + Real.1 + Real.1), r2_rot90(u))
        r2_smul_smul(s, (Real.1 + Real.1), r2_rot90(w))
        r2_smul(s, r2_smul((Real.1 + Real.1), r2_rot90(w))) =
            r2_smul(s * (Real.1 + Real.1), r2_rot90(w))
        r2_smul(s * (Real.1 + Real.1 + Real.1 + Real.1), r2_rot90(u)) =
            r2_smul((Real.1 + Real.1 + Real.1 + Real.1), r2_smul(s, r2_rot90(u)))
        r2_smul(s * (Real.1 + Real.1), r2_rot90(w)) =
            r2_smul((Real.1 + Real.1), r2_smul(s, r2_rot90(w)))
        r2_add(
            r2_smul(s, r2_rot90(r2_sub(u, w))),
            r2_smul(s * (Real.1 + Real.1 + Real.1), r2_rot90(r2_add(u, w)))) =
            r2_add(
                r2_smul((Real.1 + Real.1 + Real.1 + Real.1), r2_smul(s, r2_rot90(u))),
                r2_smul((Real.1 + Real.1), r2_smul(s, r2_rot90(w))))
        // assemble: F + s J(F) = 6w + 4 s J u + 2 s J w
        r2_add(
            r2_add(r2_smul((Real.1 + Real.1 + Real.1), r2_add(u, w)), r2_smul(s, r2_rot90(r2_sub(u, w)))),
            r2_add(
                r2_smul(s * (Real.1 + Real.1 + Real.1), r2_rot90(r2_add(u, w))),
                r2_neg(r2_smul((Real.1 + Real.1 + Real.1), r2_sub(u, w))))) =
            r2_add(
                r2_smul((Real.1 + Real.1) * (Real.1 + Real.1 + Real.1), w),
                r2_add(
                    r2_smul((Real.1 + Real.1 + Real.1 + Real.1), r2_smul(s, r2_rot90(u))),
                    r2_smul((Real.1 + Real.1), r2_smul(s, r2_rot90(w)))))
        napoleon_c(s, u, w) = r2_neg(r2_add(
            r2_smul((Real.1 + Real.1) * (Real.1 + Real.1 + Real.1), w),
            r2_add(
                r2_smul((Real.1 + Real.1 + Real.1 + Real.1), r2_smul(s, r2_rot90(u))),
                r2_smul((Real.1 + Real.1), r2_smul(s, r2_rot90(w))))))
        r2_neg(r2_add(
            r2_add(r2_smul((Real.1 + Real.1 + Real.1), r2_add(u, w)), r2_smul(s, r2_rot90(r2_sub(u, w)))),
            r2_smul(s, r2_rot90(r2_add(
                r2_smul((Real.1 + Real.1 + Real.1), r2_add(u, w)),
                r2_smul(s, r2_rot90(r2_sub(u, w)))))))) =
            napoleon_c(s, u, w)
    }
}

/// The doubled difference `V2 = 3(w + (c - a)) + s J(w - (c - a))` and
/// `V3 = 3((c - a) + u) + s J((c - a) - u)` with `c - a = -(u + w)` are the
/// images of `V1` under the 120-degree rotation:
/// `D120(V1) = 2 V3` and `D120(V3) = 2 V2`.
///
/// The two identities are expanded below through the common closed form
/// `C(u, w)`; both sides of each identity expand to `C(u, w)`.
theorem napoleon_rot_form2(s: Real, u: Pair[Real, Real], w: Pair[Real, Real]) {
    s * s = Real.1 + Real.1 + Real.1 implies
    r2_add(
        r2_add(
            r2_smul((Real.1 + Real.1 + Real.1), r2_add(r2_neg(r2_add(u, w)), u)),
            r2_smul(s, r2_rot90(r2_sub(r2_neg(r2_add(u, w)), u)))),
        r2_add(
            r2_smul((Real.1 + Real.1 + Real.1), r2_add(r2_neg(r2_add(u, w)), u)),
            r2_smul(s, r2_rot90(r2_sub(r2_neg(r2_add(u, w)), u))))) =
        napoleon_c(s, u, w)
} by {
    if s * s = Real.1 + Real.1 + Real.1 {
        s * s = Real.1 + Real.1 + Real.1
        // -(u + w) + u = -w
        r2_neg_add(u, w)
        r2_neg(r2_add(u, w)) = r2_add(r2_neg(u), r2_neg(w))
        r2_add(r2_add(r2_neg(u), r2_neg(w)), u) =
            r2_add(r2_add(r2_neg(u), u), r2_neg(w))
        r2_add_comm(r2_neg(u), u)
        r2_add(r2_neg(u), u) = r2_add(u, r2_neg(u))
        r2_add_neg_right(u)
        r2_add(u, r2_neg(u)) = Pair.new(Real.0, Real.0)
        r2_add(r2_neg(u), u) = Pair.new(Real.0, Real.0)
        r2_add_zero_left(r2_neg(w))
        r2_add(Pair.new(Real.0, Real.0), r2_neg(w)) = r2_neg(w)
        r2_add(r2_neg(r2_add(u, w)), u) = r2_neg(w)
        // -(u + w) - u = -(2u + w)
        r2_sub_eq_add_neg(r2_neg(r2_add(u, w)), u)
        r2_sub(r2_neg(r2_add(u, w)), u) = r2_add(r2_neg(r2_add(u, w)), r2_neg(u))
        r2_neg_add(r2_add(u, w), u)
        r2_neg(r2_add(r2_add(u, w), u)) = r2_add(r2_neg(r2_add(u, w)), r2_neg(u))
        r2_add(r2_add(u, w), u) = r2_add(r2_add(u, u), w)
        r2_smul_double(u)
        r2_smul((Real.1 + Real.1), u) = r2_add(u, u)
        r2_add(r2_add(u, u), w) = r2_add(r2_smul((Real.1 + Real.1), u), w)
        r2_add(r2_add(u, w), u) = r2_add(r2_smul((Real.1 + Real.1), u), w)
        r2_sub(r2_neg(r2_add(u, w)), u) = r2_neg(r2_add(r2_smul((Real.1 + Real.1), u), w))
        // F(-(u+w), u) = -3w - s J(2u + w)
        r2_add(r2_neg(r2_add(u, w)), u) = r2_neg(w)
        r2_sub(r2_neg(r2_add(u, w)), u) = r2_neg(r2_add(r2_smul((Real.1 + Real.1), u), w))
        r2_smul_neg((Real.1 + Real.1 + Real.1), w)
        r2_smul((Real.1 + Real.1 + Real.1), r2_neg(w)) = r2_neg(r2_smul((Real.1 + Real.1 + Real.1), w))
        r2_rot90_neg(r2_add(r2_smul((Real.1 + Real.1), u), w))
        r2_rot90(r2_neg(r2_add(r2_smul((Real.1 + Real.1), u), w))) =
            r2_neg(r2_rot90(r2_add(r2_smul((Real.1 + Real.1), u), w)))
        r2_rot90_add(r2_smul((Real.1 + Real.1), u), w)
        r2_rot90(r2_add(r2_smul((Real.1 + Real.1), u), w)) =
            r2_add(r2_rot90(r2_smul((Real.1 + Real.1), u)), r2_rot90(w))
        r2_rot90_smul((Real.1 + Real.1), u)
        r2_rot90(r2_smul((Real.1 + Real.1), u)) = r2_smul((Real.1 + Real.1), r2_rot90(u))
        r2_rot90(r2_add(r2_smul((Real.1 + Real.1), u), w)) =
            r2_add(r2_smul((Real.1 + Real.1), r2_rot90(u)), r2_rot90(w))
        r2_smul(s, r2_rot90(r2_neg(r2_add(r2_smul((Real.1 + Real.1), u), w)))) =
            r2_smul(s, r2_neg(r2_add(r2_smul((Real.1 + Real.1), r2_rot90(u)), r2_rot90(w))))
        r2_smul_neg(s, r2_add(r2_smul((Real.1 + Real.1), r2_rot90(u)), r2_rot90(w)))
        r2_smul(s, r2_neg(r2_add(r2_smul((Real.1 + Real.1), r2_rot90(u)), r2_rot90(w)))) =
            r2_neg(r2_smul(s, r2_add(r2_smul((Real.1 + Real.1), r2_rot90(u)), r2_rot90(w))))
        r2_smul_add_distrib(
            s, r2_smul((Real.1 + Real.1), r2_rot90(u)), r2_rot90(w))
        r2_smul(s, r2_add(r2_smul((Real.1 + Real.1), r2_rot90(u)), r2_rot90(w))) =
            r2_add(
                r2_smul(s, r2_smul((Real.1 + Real.1), r2_rot90(u))),
                r2_smul(s, r2_rot90(w)))
        r2_smul_smul(s, (Real.1 + Real.1), r2_rot90(u))
        r2_smul(s, r2_smul((Real.1 + Real.1), r2_rot90(u))) =
            r2_smul(s * (Real.1 + Real.1), r2_rot90(u))
        r2_smul(s * (Real.1 + Real.1), r2_rot90(u)) =
            r2_smul((Real.1 + Real.1), r2_smul(s, r2_rot90(u)))
        r2_smul(s, r2_add(r2_smul((Real.1 + Real.1), r2_rot90(u)), r2_rot90(w))) =
            r2_add(
                r2_smul((Real.1 + Real.1), r2_smul(s, r2_rot90(u))),
                r2_smul(s, r2_rot90(w)))
        r2_smul(s, r2_rot90(r2_neg(r2_add(r2_smul((Real.1 + Real.1), u), w)))) =
            r2_neg(r2_add(
                r2_smul((Real.1 + Real.1), r2_smul(s, r2_rot90(u))),
                r2_smul(s, r2_rot90(w))))
        // F = -3w + s J(-(2u + w)) = -(3w + 2 s J u + s J w)
        r2_add(
            r2_smul((Real.1 + Real.1 + Real.1), r2_add(r2_neg(r2_add(u, w)), u)),
            r2_smul(s, r2_rot90(r2_sub(r2_neg(r2_add(u, w)), u)))) =
            r2_add(
                r2_neg(r2_smul((Real.1 + Real.1 + Real.1), w)),
                r2_smul(s, r2_rot90(r2_neg(r2_add(r2_smul((Real.1 + Real.1), u), w)))))
        r2_add(
            r2_neg(r2_smul((Real.1 + Real.1 + Real.1), w)),
            r2_neg(r2_add(
                r2_smul((Real.1 + Real.1), r2_smul(s, r2_rot90(u))),
                r2_smul(s, r2_rot90(w))))) =
            r2_neg(r2_add(
                r2_smul((Real.1 + Real.1 + Real.1), w),
                r2_add(
                    r2_smul((Real.1 + Real.1), r2_smul(s, r2_rot90(u))),
                    r2_smul(s, r2_rot90(w)))))
        r2_neg_add(
            r2_smul((Real.1 + Real.1 + Real.1), w),
            r2_add(
                r2_smul((Real.1 + Real.1), r2_smul(s, r2_rot90(u))),
                r2_smul(s, r2_rot90(w))))
        r2_neg(r2_add(
            r2_smul((Real.1 + Real.1 + Real.1), w),
            r2_add(
                r2_smul((Real.1 + Real.1), r2_smul(s, r2_rot90(u))),
                r2_smul(s, r2_rot90(w))))) =
            r2_add(
                r2_neg(r2_smul((Real.1 + Real.1 + Real.1), w)),
                r2_neg(r2_add(
                    r2_smul((Real.1 + Real.1), r2_smul(s, r2_rot90(u))),
                    r2_smul(s, r2_rot90(w)))))
        // 2F = 2 (-(3w + 2 s J u + s J w)) = -(6w + 4 s J u + 2 s J w)
        r2_add(
            r2_add(
                r2_smul((Real.1 + Real.1 + Real.1), r2_add(r2_neg(r2_add(u, w)), u)),
                r2_smul(s, r2_rot90(r2_sub(r2_neg(r2_add(u, w)), u)))),
            r2_add(
                r2_smul((Real.1 + Real.1 + Real.1), r2_add(r2_neg(r2_add(u, w)), u)),
                r2_smul(s, r2_rot90(r2_sub(r2_neg(r2_add(u, w)), u))))) =
            r2_add(
                r2_neg(r2_add(
                    r2_smul((Real.1 + Real.1 + Real.1), w),
                    r2_add(
                        r2_smul((Real.1 + Real.1), r2_smul(s, r2_rot90(u))),
                        r2_smul(s, r2_rot90(w))))),
                r2_neg(r2_add(
                    r2_smul((Real.1 + Real.1 + Real.1), w),
                    r2_add(
                        r2_smul((Real.1 + Real.1), r2_smul(s, r2_rot90(u))),
                        r2_smul(s, r2_rot90(w))))))
        r2_neg_add(
            r2_add(
                r2_smul((Real.1 + Real.1 + Real.1), w),
                r2_add(
                    r2_smul((Real.1 + Real.1), r2_smul(s, r2_rot90(u))),
                    r2_smul(s, r2_rot90(w)))),
            r2_add(
                r2_smul((Real.1 + Real.1 + Real.1), w),
                r2_add(
                    r2_smul((Real.1 + Real.1), r2_smul(s, r2_rot90(u))),
                    r2_smul(s, r2_rot90(w)))))
        r2_neg(r2_add(
            r2_add(
                r2_smul((Real.1 + Real.1 + Real.1), w),
                r2_add(
                    r2_smul((Real.1 + Real.1), r2_smul(s, r2_rot90(u))),
                    r2_smul(s, r2_rot90(w)))),
            r2_add(
                r2_smul((Real.1 + Real.1 + Real.1), w),
                r2_add(
                    r2_smul((Real.1 + Real.1), r2_smul(s, r2_rot90(u))),
                    r2_smul(s, r2_rot90(w)))))) =
            r2_add(
                r2_neg(r2_add(
                    r2_smul((Real.1 + Real.1 + Real.1), w),
                    r2_add(
                        r2_smul((Real.1 + Real.1), r2_smul(s, r2_rot90(u))),
                        r2_smul(s, r2_rot90(w))))),
                r2_neg(r2_add(
                    r2_smul((Real.1 + Real.1 + Real.1), w),
                    r2_add(
                        r2_smul((Real.1 + Real.1), r2_smul(s, r2_rot90(u))),
                        r2_smul(s, r2_rot90(w))))))
        r2_smul_double(r2_add(
            r2_smul((Real.1 + Real.1 + Real.1), w),
            r2_add(
                r2_smul((Real.1 + Real.1), r2_smul(s, r2_rot90(u))),
                r2_smul(s, r2_rot90(w)))))
        r2_smul((Real.1 + Real.1), r2_add(
            r2_smul((Real.1 + Real.1 + Real.1), w),
            r2_add(
                r2_smul((Real.1 + Real.1), r2_smul(s, r2_rot90(u))),
                r2_smul(s, r2_rot90(w))))) =
            r2_add(
                r2_add(
                    r2_smul((Real.1 + Real.1 + Real.1), w),
                    r2_add(
                        r2_smul((Real.1 + Real.1), r2_smul(s, r2_rot90(u))),
                        r2_smul(s, r2_rot90(w)))),
                r2_add(
                    r2_smul((Real.1 + Real.1 + Real.1), w),
                    r2_add(
                        r2_smul((Real.1 + Real.1), r2_smul(s, r2_rot90(u))),
                        r2_smul(s, r2_rot90(w)))))
        r2_smul_add_distrib(
            (Real.1 + Real.1),
            r2_smul((Real.1 + Real.1 + Real.1), w),
            r2_add(
                r2_smul((Real.1 + Real.1), r2_smul(s, r2_rot90(u))),
                r2_smul(s, r2_rot90(w))))
        r2_smul((Real.1 + Real.1), r2_add(
            r2_smul((Real.1 + Real.1 + Real.1), w),
            r2_add(
                r2_smul((Real.1 + Real.1), r2_smul(s, r2_rot90(u))),
                r2_smul(s, r2_rot90(w))))) =
            r2_add(
                r2_smul((Real.1 + Real.1), r2_smul((Real.1 + Real.1 + Real.1), w)),
                r2_smul((Real.1 + Real.1), r2_add(
                    r2_smul((Real.1 + Real.1), r2_smul(s, r2_rot90(u))),
                    r2_smul(s, r2_rot90(w)))))
        r2_smul_smul((Real.1 + Real.1), (Real.1 + Real.1 + Real.1), w)
        r2_smul((Real.1 + Real.1), r2_smul((Real.1 + Real.1 + Real.1), w)) =
            r2_smul((Real.1 + Real.1) * (Real.1 + Real.1 + Real.1), w)
        r2_smul_add_distrib(
            (Real.1 + Real.1),
            r2_smul((Real.1 + Real.1), r2_smul(s, r2_rot90(u))),
            r2_smul(s, r2_rot90(w)))
        r2_smul((Real.1 + Real.1), r2_add(
            r2_smul((Real.1 + Real.1), r2_smul(s, r2_rot90(u))),
            r2_smul(s, r2_rot90(w)))) =
            r2_add(
                r2_smul((Real.1 + Real.1), r2_smul((Real.1 + Real.1), r2_smul(s, r2_rot90(u)))),
                r2_smul((Real.1 + Real.1), r2_smul(s, r2_rot90(w))))
        r2_smul_smul((Real.1 + Real.1), (Real.1 + Real.1), r2_smul(s, r2_rot90(u)))
        r2_smul((Real.1 + Real.1), r2_smul((Real.1 + Real.1), r2_smul(s, r2_rot90(u)))) =
            r2_smul((Real.1 + Real.1) * (Real.1 + Real.1), r2_smul(s, r2_rot90(u)))
        r2_smul((Real.1 + Real.1 + Real.1 + Real.1), r2_smul(s, r2_rot90(u))) =
            r2_smul((Real.1 + Real.1) * (Real.1 + Real.1), r2_smul(s, r2_rot90(u)))
        r2_add(
            r2_add(
                r2_smul((Real.1 + Real.1 + Real.1), r2_add(r2_neg(r2_add(u, w)), u)),
                r2_smul(s, r2_rot90(r2_sub(r2_neg(r2_add(u, w)), u)))),
            r2_add(
                r2_smul((Real.1 + Real.1 + Real.1), r2_add(r2_neg(r2_add(u, w)), u)),
                r2_smul(s, r2_rot90(r2_sub(r2_neg(r2_add(u, w)), u))))) =
            r2_neg(r2_add(
                r2_smul((Real.1 + Real.1) * (Real.1 + Real.1 + Real.1), w),
                r2_add(
                    r2_smul((Real.1 + Real.1 + Real.1 + Real.1), r2_smul(s, r2_rot90(u))),
                    r2_smul((Real.1 + Real.1), r2_smul(s, r2_rot90(w))))))
        napoleon_c(s, u, w) = r2_neg(r2_add(
            r2_smul((Real.1 + Real.1) * (Real.1 + Real.1 + Real.1), w),
            r2_add(
                r2_smul((Real.1 + Real.1 + Real.1 + Real.1), r2_smul(s, r2_rot90(u))),
                r2_smul((Real.1 + Real.1), r2_smul(s, r2_rot90(w))))))
        r2_add(
            r2_add(
                r2_smul((Real.1 + Real.1 + Real.1), r2_add(r2_neg(r2_add(u, w)), u)),
                r2_smul(s, r2_rot90(r2_sub(r2_neg(r2_add(u, w)), u)))),
            r2_add(
                r2_smul((Real.1 + Real.1 + Real.1), r2_add(r2_neg(r2_add(u, w)), u)),
                r2_smul(s, r2_rot90(r2_sub(r2_neg(r2_add(u, w)), u))))) =
            napoleon_c(s, u, w)
    }
}

// ---------------------------------------------------------------------------
// The cyclic pair identities: `c - a = -(u + w)`, `-(w + (-(u + w))) = u`.
// ---------------------------------------------------------------------------

/// `-(a - b + b - c) = c - a`.
theorem napoleon_pair1(
    a: Pair[Real, Real], b: Pair[Real, Real], c: Pair[Real, Real]
) {
    r2_neg(r2_add(r2_sub(a, b), r2_sub(b, c))) = r2_sub(c, a)
} by {
    r2_neg_add(r2_sub(a, b), r2_sub(b, c))
    r2_neg(r2_add(r2_sub(a, b), r2_sub(b, c))) =
        r2_add(r2_neg(r2_sub(a, b)), r2_neg(r2_sub(b, c)))
    r2_sub_reverse_neg(a, b)
    r2_sub(a, b) = r2_neg(r2_sub(b, a))
    r2_neg(r2_sub(a, b)) = r2_sub(b, a)
    r2_sub_reverse_neg(b, c)
    r2_sub(b, c) = r2_neg(r2_sub(c, b))
    r2_neg(r2_sub(b, c)) = r2_sub(c, b)
    r2_add(r2_neg(r2_sub(a, b)), r2_neg(r2_sub(b, c))) =
        r2_add(r2_sub(b, a), r2_sub(c, b))
    r2_sub_through(b, a, c)
    r2_sub(c, a) = r2_add(r2_sub(b, a), r2_sub(c, b))
    r2_add(r2_sub(b, a), r2_sub(c, b)) = r2_sub(c, a)
    r2_neg(r2_add(r2_sub(a, b), r2_sub(b, c))) = r2_sub(c, a)
}

/// `-((b - c) + (-(a - b + b - c))) = a - b`.
theorem napoleon_pair2(
    a: Pair[Real, Real], b: Pair[Real, Real], c: Pair[Real, Real]
) {
    r2_neg(r2_add(
        r2_sub(b, c),
        r2_neg(r2_add(r2_sub(a, b), r2_sub(b, c))))) = r2_sub(a, b)
} by {
    r2_neg_add(
        r2_sub(b, c),
        r2_neg(r2_add(r2_sub(a, b), r2_sub(b, c))))
    r2_neg(r2_add(
        r2_sub(b, c),
        r2_neg(r2_add(r2_sub(a, b), r2_sub(b, c))))) =
        r2_add(
            r2_neg(r2_sub(b, c)),
            r2_neg(r2_neg(r2_add(r2_sub(a, b), r2_sub(b, c)))))
    r2_neg_neg(r2_add(r2_sub(a, b), r2_sub(b, c)))
    r2_neg(r2_neg(r2_add(r2_sub(a, b), r2_sub(b, c)))) =
        r2_add(r2_sub(a, b), r2_sub(b, c))
    r2_add(
        r2_neg(r2_sub(b, c)),
        r2_neg(r2_neg(r2_add(r2_sub(a, b), r2_sub(b, c))))) =
        r2_add(r2_neg(r2_sub(b, c)), r2_add(r2_sub(a, b), r2_sub(b, c)))
    r2_sub_reverse_neg(b, c)
    r2_neg(r2_sub(b, c)) = r2_sub(c, b)
    r2_add(r2_neg(r2_sub(b, c)), r2_add(r2_sub(a, b), r2_sub(b, c))) =
        r2_add(r2_sub(c, b), r2_add(r2_sub(a, b), r2_sub(b, c)))
    r2_add_pair_rearrange(r2_sub(a, b), r2_sub(b, c), r2_sub(c, b), Pair.new(Real.0, Real.0))
    r2_add(r2_add(r2_sub(a, b), r2_sub(b, c)), r2_add(r2_sub(c, b), Pair.new(Real.0, Real.0))) =
        r2_add(r2_add(r2_sub(a, b), r2_sub(c, b)), r2_add(r2_sub(b, c), Pair.new(Real.0, Real.0)))
    r2_add_zero_right(r2_sub(c, b))
    r2_add(r2_sub(c, b), Pair.new(Real.0, Real.0)) = r2_sub(c, b)
    r2_add(r2_add(r2_sub(a, b), r2_sub(b, c)), r2_add(r2_sub(c, b), Pair.new(Real.0, Real.0))) =
        r2_add(r2_add(r2_sub(a, b), r2_sub(b, c)), r2_sub(c, b))
    r2_add_comm(r2_sub(c, b), r2_add(r2_sub(a, b), r2_sub(b, c)))
    r2_add(r2_sub(c, b), r2_add(r2_sub(a, b), r2_sub(b, c))) =
        r2_add(r2_add(r2_sub(a, b), r2_sub(b, c)), r2_sub(c, b))
    r2_sub_reverse_neg(a, b)
    r2_sub(a, b) = r2_neg(r2_sub(b, a))
    r2_neg(r2_sub(a, b)) = r2_sub(b, a)
    r2_sub_reverse_neg(b, a)
    r2_sub(b, a) = r2_neg(r2_sub(a, b))
    r2_sub(c, b) = r2_neg(r2_sub(b, c))
    r2_add(r2_sub(b, c), r2_sub(c, b)) = Pair.new(Real.0, Real.0)
    r2_add(r2_sub(a, b), r2_add(r2_sub(b, c), r2_sub(c, b))) =
        r2_add(r2_add(r2_sub(a, b), r2_sub(b, c)), r2_sub(c, b))
    r2_add(r2_add(r2_sub(a, b), r2_sub(b, c)), r2_sub(c, b)) =
        r2_add(r2_sub(a, b), r2_add(r2_sub(b, c), r2_sub(c, b)))
    r2_add(r2_sub(b, c), r2_sub(c, b)) = Pair.new(Real.0, Real.0)
    r2_add(r2_sub(a, b), r2_add(r2_sub(b, c), r2_sub(c, b))) =
        r2_add(r2_sub(a, b), Pair.new(Real.0, Real.0))
    r2_add_zero_right(r2_sub(a, b))
    r2_add(r2_sub(a, b), Pair.new(Real.0, Real.0)) = r2_sub(a, b)
    r2_add(r2_add(r2_sub(a, b), r2_sub(b, c)), r2_sub(c, b)) = r2_sub(a, b)
    r2_add(r2_sub(c, b), r2_add(r2_sub(a, b), r2_sub(b, c))) = r2_sub(a, b)
    r2_neg(r2_add(
        r2_sub(b, c),
        r2_neg(r2_add(r2_sub(a, b), r2_sub(b, c))))) = r2_sub(a, b)
}

// ---------------------------------------------------------------------------
// Cancellation of common nonzero multiples.
// ---------------------------------------------------------------------------

/// A quadrupled real equality cancels.
theorem napoleon_cancel4(x: Real, y: Real) {
    (Real.1 + Real.1) * ((Real.1 + Real.1) * x) =
        (Real.1 + Real.1) * ((Real.1 + Real.1) * y)
    implies x = y
} by {
    if (Real.1 + Real.1) * ((Real.1 + Real.1) * x) =
        (Real.1 + Real.1) * ((Real.1 + Real.1) * y) {
        real_double_eq_add_self(x)
        real_double_eq_add_self(y)
        real_double_eq_add_self((Real.1 + Real.1) * x)
        real_double_eq_add_self((Real.1 + Real.1) * y)
        (Real.1 + Real.1) * ((Real.1 + Real.1) * x) = ((Real.1 + Real.1) * x) + ((Real.1 + Real.1) * x)
        (Real.1 + Real.1) * ((Real.1 + Real.1) * y) = ((Real.1 + Real.1) * y) + ((Real.1 + Real.1) * y)
        ((Real.1 + Real.1) * x) + ((Real.1 + Real.1) * x) =
            ((Real.1 + Real.1) * y) + ((Real.1 + Real.1) * y)
        real_double_eq_double((Real.1 + Real.1) * x, (Real.1 + Real.1) * y)
        (Real.1 + Real.1) * x = (Real.1 + Real.1) * y
        (Real.1 + Real.1) * x = x + x
        (Real.1 + Real.1) * y = y + y
        x + x = y + y
        real_double_eq_double(x, y)
        x = y
    }
}

/// A common nonzero multiple cancels from a real equality.
theorem napoleon_cancel(x: Real, y: Real) {
    (Real.1 + Real.1) * (Real.1 + Real.1 + Real.1) *
        ((Real.1 + Real.1) * (Real.1 + Real.1 + Real.1) * x) =
        (Real.1 + Real.1) * (Real.1 + Real.1 + Real.1) *
        ((Real.1 + Real.1) * (Real.1 + Real.1 + Real.1) * y)
    implies x = y
} by {
    if (Real.1 + Real.1) * (Real.1 + Real.1 + Real.1) *
        ((Real.1 + Real.1) * (Real.1 + Real.1 + Real.1) * x) =
        (Real.1 + Real.1) * (Real.1 + Real.1 + Real.1) *
        ((Real.1 + Real.1) * (Real.1 + Real.1 + Real.1) * y) {
        real_two_neq_zero
        Real.1 + Real.1 != Real.0
        Real.0 < Real.1
        Real.0 + Real.0 < Real.1 + Real.1
        Real.0 < Real.1 + Real.1
        Real.0 + Real.0 < (Real.1 + Real.1) + Real.1
        Real.0 < Real.1 + Real.1 + Real.1
        Real.1 + Real.1 + Real.1 != Real.0
        (Real.1 + Real.1) * (Real.1 + Real.1 + Real.1) != Real.0
        mul_left_cancel(
            (Real.1 + Real.1) * (Real.1 + Real.1 + Real.1) *
                ((Real.1 + Real.1) * (Real.1 + Real.1 + Real.1) * x),
            (Real.1 + Real.1) * (Real.1 + Real.1 + Real.1),
            (Real.1 + Real.1) * (Real.1 + Real.1 + Real.1) * y)
        ((Real.1 + Real.1) * (Real.1 + Real.1 + Real.1) *
            ((Real.1 + Real.1) * (Real.1 + Real.1 + Real.1) * x)) /
            ((Real.1 + Real.1) * (Real.1 + Real.1 + Real.1)) =
            (Real.1 + Real.1) * (Real.1 + Real.1 + Real.1) * y
        div_mul_cancel_left(
            (Real.1 + Real.1) * (Real.1 + Real.1 + Real.1),
            (Real.1 + Real.1) * (Real.1 + Real.1 + Real.1) * x)
        ((Real.1 + Real.1) * (Real.1 + Real.1 + Real.1) *
            ((Real.1 + Real.1) * (Real.1 + Real.1 + Real.1) * x)) /
            ((Real.1 + Real.1) * (Real.1 + Real.1 + Real.1)) =
            (Real.1 + Real.1) * (Real.1 + Real.1 + Real.1) * x
        (Real.1 + Real.1) * (Real.1 + Real.1 + Real.1) * x =
            (Real.1 + Real.1) * (Real.1 + Real.1 + Real.1) * y
        mul_left_cancel(
            (Real.1 + Real.1) * (Real.1 + Real.1 + Real.1) * x,
            (Real.1 + Real.1) * (Real.1 + Real.1 + Real.1), y)
        ((Real.1 + Real.1) * (Real.1 + Real.1 + Real.1) * x) /
            ((Real.1 + Real.1) * (Real.1 + Real.1 + Real.1)) = y
        div_mul_cancel_left((Real.1 + Real.1) * (Real.1 + Real.1 + Real.1), x)
        ((Real.1 + Real.1) * (Real.1 + Real.1 + Real.1) * x) /
            ((Real.1 + Real.1) * (Real.1 + Real.1 + Real.1)) = x
        x = y
    }
}

// ---------------------------------------------------------------------------
// Napoleon's theorem.
// ---------------------------------------------------------------------------

/// Napoleon's theorem.
///
/// With `n1`, `n2`, `n3` the centers of the equilateral triangles erected on
/// the sides `ab`, `bc`, `ca` of the triangle `abc` (the centers satisfy
/// `6 n1 = 3 (a + b) - s J(b - a)` and cyclically, where `s * s = 3` and `J`
/// is the quarter turn), the triangle `n1 n2 n3` is equilateral:
/// `|n1 - n2|^2 = |n2 - n3|^2 = |n3 - n1|^2`.
theorem theorems1000_napoleon(
    s: Real,
    a: Pair[Real, Real], b: Pair[Real, Real], c: Pair[Real, Real],
    n1: Pair[Real, Real], n2: Pair[Real, Real], n3: Pair[Real, Real]
) {
    s * s = Real.1 + Real.1 + Real.1 and
    r2_smul((Real.1 + Real.1) * (Real.1 + Real.1 + Real.1), n1) =
        r2_sub(r2_smul((Real.1 + Real.1 + Real.1), r2_add(a, b)), r2_smul(s, r2_rot90(r2_sub(b, a)))) and
    r2_smul((Real.1 + Real.1) * (Real.1 + Real.1 + Real.1), n2) =
        r2_sub(r2_smul((Real.1 + Real.1 + Real.1), r2_add(b, c)), r2_smul(s, r2_rot90(r2_sub(c, b)))) and
    r2_smul((Real.1 + Real.1) * (Real.1 + Real.1 + Real.1), n3) =
        r2_sub(r2_smul((Real.1 + Real.1 + Real.1), r2_add(c, a)), r2_smul(s, r2_rot90(r2_sub(a, c))))
    implies
    r2_norm_sq(r2_sub(n1, n2)) = r2_norm_sq(r2_sub(n2, n3)) and
    r2_norm_sq(r2_sub(n2, n3)) = r2_norm_sq(r2_sub(n3, n1))
} by {
    if s * s = Real.1 + Real.1 + Real.1 and
        r2_smul((Real.1 + Real.1) * (Real.1 + Real.1 + Real.1), n1) =
            r2_sub(r2_smul((Real.1 + Real.1 + Real.1), r2_add(a, b)), r2_smul(s, r2_rot90(r2_sub(b, a)))) and
        r2_smul((Real.1 + Real.1) * (Real.1 + Real.1 + Real.1), n2) =
            r2_sub(r2_smul((Real.1 + Real.1 + Real.1), r2_add(b, c)), r2_smul(s, r2_rot90(r2_sub(c, b)))) and
        r2_smul((Real.1 + Real.1) * (Real.1 + Real.1 + Real.1), n3) =
            r2_sub(r2_smul((Real.1 + Real.1 + Real.1), r2_add(c, a)), r2_smul(s, r2_rot90(r2_sub(a, c)))) {
        s * s = Real.1 + Real.1 + Real.1
        r2_smul((Real.1 + Real.1) * (Real.1 + Real.1 + Real.1), n1) =
            r2_sub(r2_smul((Real.1 + Real.1 + Real.1), r2_add(a, b)), r2_smul(s, r2_rot90(r2_sub(b, a))))
        r2_smul((Real.1 + Real.1) * (Real.1 + Real.1 + Real.1), n2) =
            r2_sub(r2_smul((Real.1 + Real.1 + Real.1), r2_add(b, c)), r2_smul(s, r2_rot90(r2_sub(c, b))))
        r2_smul((Real.1 + Real.1) * (Real.1 + Real.1 + Real.1), n3) =
            r2_sub(r2_smul((Real.1 + Real.1 + Real.1), r2_add(c, a)), r2_smul(s, r2_rot90(r2_sub(a, c))))
        // V1 = 6(n1 - n2) = F(a - b, b - c)
        napoleon_six_sub(a, b, c, s, n1, n2, r2_sub(a, b), r2_sub(b, c))
        r2_smul((Real.1 + Real.1) * (Real.1 + Real.1 + Real.1), r2_sub(n1, n2)) =
            r2_add(
                r2_smul((Real.1 + Real.1 + Real.1), r2_add(r2_sub(a, b), r2_sub(b, c))),
                r2_smul(s, r2_rot90(r2_sub(r2_sub(a, b), r2_sub(b, c)))))
        // V3 = 6(n3 - n1) = F(c - a, a - b)
        napoleon_six_sub(c, a, b, s, n3, n1, r2_sub(c, a), r2_sub(a, b))
        r2_smul((Real.1 + Real.1) * (Real.1 + Real.1 + Real.1), r2_sub(n3, n1)) =
            r2_add(
                r2_smul((Real.1 + Real.1 + Real.1), r2_add(r2_sub(c, a), r2_sub(a, b))),
                r2_smul(s, r2_rot90(r2_sub(r2_sub(c, a), r2_sub(a, b)))))
        // V2 = 6(n2 - n3) = F(b - c, c - a)
        napoleon_six_sub(b, c, a, s, n2, n3, r2_sub(b, c), r2_sub(c, a))
        r2_smul((Real.1 + Real.1) * (Real.1 + Real.1 + Real.1), r2_sub(n2, n3)) =
            r2_add(
                r2_smul((Real.1 + Real.1 + Real.1), r2_add(r2_sub(b, c), r2_sub(c, a))),
                r2_smul(s, r2_rot90(r2_sub(r2_sub(b, c), r2_sub(c, a)))))
        // c - a = -(a - b + b - c)
        napoleon_pair1(a, b, c)
        r2_neg(r2_add(r2_sub(a, b), r2_sub(b, c))) = r2_sub(c, a)
        // F(c - a, a - b) = F(-(a - b + b - c), a - b)
        r2_add(r2_sub(c, a), r2_sub(a, b)) =
            r2_add(r2_neg(r2_add(r2_sub(a, b), r2_sub(b, c))), r2_sub(a, b))
        r2_sub(r2_sub(c, a), r2_sub(a, b)) =
            r2_sub(r2_neg(r2_add(r2_sub(a, b), r2_sub(b, c))), r2_sub(a, b))
        r2_smul((Real.1 + Real.1 + Real.1), r2_add(r2_sub(c, a), r2_sub(a, b))) =
            r2_smul((Real.1 + Real.1 + Real.1), r2_add(r2_neg(r2_add(r2_sub(a, b), r2_sub(b, c))), r2_sub(a, b)))
        r2_rot90(r2_sub(r2_sub(c, a), r2_sub(a, b))) =
            r2_rot90(r2_sub(r2_neg(r2_add(r2_sub(a, b), r2_sub(b, c))), r2_sub(a, b)))
        r2_smul(s, r2_rot90(r2_sub(r2_sub(c, a), r2_sub(a, b)))) =
            r2_smul(s, r2_rot90(r2_sub(r2_neg(r2_add(r2_sub(a, b), r2_sub(b, c))), r2_sub(a, b))))
        r2_add(
            r2_smul((Real.1 + Real.1 + Real.1), r2_add(r2_sub(c, a), r2_sub(a, b))),
            r2_smul(s, r2_rot90(r2_sub(r2_sub(c, a), r2_sub(a, b))))) =
            r2_add(
                r2_smul((Real.1 + Real.1 + Real.1), r2_add(r2_neg(r2_add(r2_sub(a, b), r2_sub(b, c))), r2_sub(a, b))),
                r2_smul(s, r2_rot90(r2_sub(r2_neg(r2_add(r2_sub(a, b), r2_sub(b, c))), r2_sub(a, b)))))
        r2_smul((Real.1 + Real.1) * (Real.1 + Real.1 + Real.1), r2_sub(n3, n1)) =
            r2_add(
                r2_smul((Real.1 + Real.1 + Real.1), r2_add(r2_neg(r2_add(r2_sub(a, b), r2_sub(b, c))), r2_sub(a, b))),
                r2_smul(s, r2_rot90(r2_sub(r2_neg(r2_add(r2_sub(a, b), r2_sub(b, c))), r2_sub(a, b)))))
        // D120(V1) = C(a - b, b - c) and 2 F(-(a - b + b - c), a - b) = C(a - b, b - c)
        napoleon_rot_form1(s, r2_sub(a, b), r2_sub(b, c))
        r2_neg(r2_add(
            r2_add(
                r2_smul((Real.1 + Real.1 + Real.1), r2_add(r2_sub(a, b), r2_sub(b, c))),
                r2_smul(s, r2_rot90(r2_sub(r2_sub(a, b), r2_sub(b, c))))),
            r2_smul(s, r2_rot90(r2_add(
                r2_smul((Real.1 + Real.1 + Real.1), r2_add(r2_sub(a, b), r2_sub(b, c))),
                r2_smul(s, r2_rot90(r2_sub(r2_sub(a, b), r2_sub(b, c))))))))) =
            napoleon_c(s, r2_sub(a, b), r2_sub(b, c))
        r2_neg(r2_add(
            r2_smul((Real.1 + Real.1) * (Real.1 + Real.1 + Real.1), r2_sub(n1, n2)),
            r2_smul(s, r2_rot90(r2_smul((Real.1 + Real.1) * (Real.1 + Real.1 + Real.1), r2_sub(n1, n2)))))) =
            napoleon_c(s, r2_sub(a, b), r2_sub(b, c))
        napoleon_rot_form2(s, r2_sub(a, b), r2_sub(b, c))
        r2_add(
            r2_add(
                r2_smul((Real.1 + Real.1 + Real.1), r2_add(r2_neg(r2_add(r2_sub(a, b), r2_sub(b, c))), r2_sub(a, b))),
                r2_smul(s, r2_rot90(r2_sub(r2_neg(r2_add(r2_sub(a, b), r2_sub(b, c))), r2_sub(a, b))))),
            r2_add(
                r2_smul((Real.1 + Real.1 + Real.1), r2_add(r2_neg(r2_add(r2_sub(a, b), r2_sub(b, c))), r2_sub(a, b))),
                r2_smul(s, r2_rot90(r2_sub(r2_neg(r2_add(r2_sub(a, b), r2_sub(b, c))), r2_sub(a, b)))))) =
            napoleon_c(s, r2_sub(a, b), r2_sub(b, c))
        r2_add(
            r2_smul((Real.1 + Real.1) * (Real.1 + Real.1 + Real.1), r2_sub(n3, n1)),
            r2_smul((Real.1 + Real.1) * (Real.1 + Real.1 + Real.1), r2_sub(n3, n1))) =
            napoleon_c(s, r2_sub(a, b), r2_sub(b, c))
        // D120(V1) = 2 V3
        r2_neg(r2_add(
            r2_smul((Real.1 + Real.1) * (Real.1 + Real.1 + Real.1), r2_sub(n1, n2)),
            r2_smul(s, r2_rot90(r2_smul((Real.1 + Real.1) * (Real.1 + Real.1 + Real.1), r2_sub(n1, n2)))))) =
            r2_add(
                r2_smul((Real.1 + Real.1) * (Real.1 + Real.1 + Real.1), r2_sub(n3, n1)),
                r2_smul((Real.1 + Real.1) * (Real.1 + Real.1 + Real.1), r2_sub(n3, n1)))
        // |D120(V1)|^2 = 4|V1|^2 and |2 V3|^2 = 4|V3|^2
        napoleon_d120_norm(s, r2_smul((Real.1 + Real.1) * (Real.1 + Real.1 + Real.1), r2_sub(n1, n2)))
        r2_norm_sq(r2_neg(r2_add(
            r2_smul((Real.1 + Real.1) * (Real.1 + Real.1 + Real.1), r2_sub(n1, n2)),
            r2_smul(s, r2_rot90(r2_smul((Real.1 + Real.1) * (Real.1 + Real.1 + Real.1), r2_sub(n1, n2))))))) =
            (Real.1 + Real.1 + Real.1 + Real.1) *
            r2_norm_sq(r2_smul((Real.1 + Real.1) * (Real.1 + Real.1 + Real.1), r2_sub(n1, n2)))
        r2_norm_sq_double(r2_smul((Real.1 + Real.1) * (Real.1 + Real.1 + Real.1), r2_sub(n3, n1)))
        r2_norm_sq(r2_add(
            r2_smul((Real.1 + Real.1) * (Real.1 + Real.1 + Real.1), r2_sub(n3, n1)),
            r2_smul((Real.1 + Real.1) * (Real.1 + Real.1 + Real.1), r2_sub(n3, n1)))) =
            (Real.1 + Real.1) * ((Real.1 + Real.1) *
            r2_norm_sq(r2_smul((Real.1 + Real.1) * (Real.1 + Real.1 + Real.1), r2_sub(n3, n1))))
        (Real.1 + Real.1 + Real.1 + Real.1) *
            r2_norm_sq(r2_smul((Real.1 + Real.1) * (Real.1 + Real.1 + Real.1), r2_sub(n1, n2))) =
            (Real.1 + Real.1) * ((Real.1 + Real.1) *
            r2_norm_sq(r2_smul((Real.1 + Real.1) * (Real.1 + Real.1 + Real.1), r2_sub(n3, n1))))
        (Real.1 + Real.1 + Real.1 + Real.1) *
            r2_norm_sq(r2_smul((Real.1 + Real.1) * (Real.1 + Real.1 + Real.1), r2_sub(n1, n2))) =
            ((Real.1 + Real.1) + (Real.1 + Real.1)) *
            r2_norm_sq(r2_smul((Real.1 + Real.1) * (Real.1 + Real.1 + Real.1), r2_sub(n1, n2)))
        ((Real.1 + Real.1) + (Real.1 + Real.1)) *
            r2_norm_sq(r2_smul((Real.1 + Real.1) * (Real.1 + Real.1 + Real.1), r2_sub(n1, n2))) =
            (Real.1 + Real.1) * r2_norm_sq(r2_smul((Real.1 + Real.1) * (Real.1 + Real.1 + Real.1), r2_sub(n1, n2))) +
            (Real.1 + Real.1) * r2_norm_sq(r2_smul((Real.1 + Real.1) * (Real.1 + Real.1 + Real.1), r2_sub(n1, n2)))
        (Real.1 + Real.1) * r2_norm_sq(r2_smul((Real.1 + Real.1) * (Real.1 + Real.1 + Real.1), r2_sub(n1, n2))) +
            (Real.1 + Real.1) * r2_norm_sq(r2_smul((Real.1 + Real.1) * (Real.1 + Real.1 + Real.1), r2_sub(n1, n2))) =
            (Real.1 + Real.1) * ((Real.1 + Real.1) *
            r2_norm_sq(r2_smul((Real.1 + Real.1) * (Real.1 + Real.1 + Real.1), r2_sub(n1, n2))))
        // |V1|^2 = |V3|^2 via cancellation of four
        (Real.1 + Real.1) * ((Real.1 + Real.1) *
            r2_norm_sq(r2_smul((Real.1 + Real.1) * (Real.1 + Real.1 + Real.1), r2_sub(n1, n2)))) =
            (Real.1 + Real.1) * ((Real.1 + Real.1) *
            r2_norm_sq(r2_smul((Real.1 + Real.1) * (Real.1 + Real.1 + Real.1), r2_sub(n3, n1))))
        napoleon_cancel4(
            r2_norm_sq(r2_smul((Real.1 + Real.1) * (Real.1 + Real.1 + Real.1), r2_sub(n1, n2))),
            r2_norm_sq(r2_smul((Real.1 + Real.1) * (Real.1 + Real.1 + Real.1), r2_sub(n3, n1))))
        r2_norm_sq(r2_smul((Real.1 + Real.1) * (Real.1 + Real.1 + Real.1), r2_sub(n1, n2))) =
            r2_norm_sq(r2_smul((Real.1 + Real.1) * (Real.1 + Real.1 + Real.1), r2_sub(n3, n1)))
        // 36|n1 - n2|^2 = 36|n3 - n1|^2, then cancel 36
        r2_norm_sq_smul((Real.1 + Real.1) * (Real.1 + Real.1 + Real.1), r2_sub(n1, n2))
        r2_norm_sq(r2_smul((Real.1 + Real.1) * (Real.1 + Real.1 + Real.1), r2_sub(n1, n2))) =
            (Real.1 + Real.1) * (Real.1 + Real.1 + Real.1) *
            ((Real.1 + Real.1) * (Real.1 + Real.1 + Real.1) * r2_norm_sq(r2_sub(n1, n2)))
        r2_norm_sq_smul((Real.1 + Real.1) * (Real.1 + Real.1 + Real.1), r2_sub(n3, n1))
        r2_norm_sq(r2_smul((Real.1 + Real.1) * (Real.1 + Real.1 + Real.1), r2_sub(n3, n1))) =
            (Real.1 + Real.1) * (Real.1 + Real.1 + Real.1) *
            ((Real.1 + Real.1) * (Real.1 + Real.1 + Real.1) * r2_norm_sq(r2_sub(n3, n1)))
        (Real.1 + Real.1) * (Real.1 + Real.1 + Real.1) *
            ((Real.1 + Real.1) * (Real.1 + Real.1 + Real.1) * r2_norm_sq(r2_sub(n1, n2))) =
            (Real.1 + Real.1) * (Real.1 + Real.1 + Real.1) *
            ((Real.1 + Real.1) * (Real.1 + Real.1 + Real.1) * r2_norm_sq(r2_sub(n3, n1)))
        napoleon_cancel(r2_norm_sq(r2_sub(n1, n2)), r2_norm_sq(r2_sub(n3, n1)))
        r2_norm_sq(r2_sub(n1, n2)) = r2_norm_sq(r2_sub(n3, n1))
        // D120(V2) = 2 V1: rotate the argument cyclically
        r2_add(r2_sub(b, c), r2_sub(c, a)) =
            r2_add(r2_sub(b, c), r2_neg(r2_add(r2_sub(a, b), r2_sub(b, c))))
        r2_sub(r2_sub(b, c), r2_sub(c, a)) =
            r2_sub(r2_sub(b, c), r2_neg(r2_add(r2_sub(a, b), r2_sub(b, c))))
        r2_smul((Real.1 + Real.1 + Real.1), r2_add(r2_sub(b, c), r2_sub(c, a))) =
            r2_smul((Real.1 + Real.1 + Real.1), r2_add(r2_sub(b, c), r2_neg(r2_add(r2_sub(a, b), r2_sub(b, c)))))
        r2_rot90(r2_sub(r2_sub(b, c), r2_sub(c, a))) =
            r2_rot90(r2_sub(r2_sub(b, c), r2_neg(r2_add(r2_sub(a, b), r2_sub(b, c)))))
        r2_smul(s, r2_rot90(r2_sub(r2_sub(b, c), r2_sub(c, a)))) =
            r2_smul(s, r2_rot90(r2_sub(r2_sub(b, c), r2_neg(r2_add(r2_sub(a, b), r2_sub(b, c))))))
        r2_smul((Real.1 + Real.1) * (Real.1 + Real.1 + Real.1), r2_sub(n2, n3)) =
            r2_add(
                r2_smul((Real.1 + Real.1 + Real.1), r2_add(r2_sub(b, c), r2_neg(r2_add(r2_sub(a, b), r2_sub(b, c))))),
                r2_smul(s, r2_rot90(r2_sub(r2_sub(b, c), r2_neg(r2_add(r2_sub(a, b), r2_sub(b, c)))))))
        napoleon_rot_form1(s, r2_sub(b, c), r2_neg(r2_add(r2_sub(a, b), r2_sub(b, c))))
        r2_neg(r2_add(
            r2_add(
                r2_smul((Real.1 + Real.1 + Real.1), r2_add(r2_sub(b, c), r2_neg(r2_add(r2_sub(a, b), r2_sub(b, c))))),
                r2_smul(s, r2_rot90(r2_sub(r2_sub(b, c), r2_neg(r2_add(r2_sub(a, b), r2_sub(b, c))))))),
            r2_smul(s, r2_rot90(r2_add(
                r2_smul((Real.1 + Real.1 + Real.1), r2_add(r2_sub(b, c), r2_neg(r2_add(r2_sub(a, b), r2_sub(b, c))))),
                r2_smul(s, r2_rot90(r2_sub(r2_sub(b, c), r2_neg(r2_add(r2_sub(a, b), r2_sub(b, c))))))))))) =
            napoleon_c(s, r2_sub(b, c), r2_neg(r2_add(r2_sub(a, b), r2_sub(b, c))))
        r2_neg(r2_add(
            r2_smul((Real.1 + Real.1) * (Real.1 + Real.1 + Real.1), r2_sub(n2, n3)),
            r2_smul(s, r2_rot90(r2_smul((Real.1 + Real.1) * (Real.1 + Real.1 + Real.1), r2_sub(n2, n3)))))) =
            napoleon_c(s, r2_sub(b, c), r2_neg(r2_add(r2_sub(a, b), r2_sub(b, c))))
        napoleon_rot_form2(s, r2_sub(b, c), r2_neg(r2_add(r2_sub(a, b), r2_sub(b, c))))
        r2_add(
            r2_add(
                r2_smul((Real.1 + Real.1 + Real.1), r2_add(
                    r2_neg(r2_add(r2_sub(b, c), r2_neg(r2_add(r2_sub(a, b), r2_sub(b, c))))),
                    r2_sub(b, c))),
                r2_smul(s, r2_rot90(r2_sub(
                    r2_neg(r2_add(r2_sub(b, c), r2_neg(r2_add(r2_sub(a, b), r2_sub(b, c))))),
                    r2_sub(b, c))))),
            r2_add(
                r2_smul((Real.1 + Real.1 + Real.1), r2_add(
                    r2_neg(r2_add(r2_sub(b, c), r2_neg(r2_add(r2_sub(a, b), r2_sub(b, c))))),
                    r2_sub(b, c))),
                r2_smul(s, r2_rot90(r2_sub(
                    r2_neg(r2_add(r2_sub(b, c), r2_neg(r2_add(r2_sub(a, b), r2_sub(b, c))))),
                    r2_sub(b, c)))))) =
            napoleon_c(s, r2_sub(b, c), r2_neg(r2_add(r2_sub(a, b), r2_sub(b, c))))
        napoleon_pair2(a, b, c)
        r2_neg(r2_add(
            r2_sub(b, c),
            r2_neg(r2_add(r2_sub(a, b), r2_sub(b, c))))) = r2_sub(a, b)
        r2_add(
            r2_neg(r2_add(r2_sub(b, c), r2_neg(r2_add(r2_sub(a, b), r2_sub(b, c))))),
            r2_sub(b, c)) =
            r2_add(r2_sub(a, b), r2_sub(b, c))
        r2_sub(
            r2_neg(r2_add(r2_sub(b, c), r2_neg(r2_add(r2_sub(a, b), r2_sub(b, c))))),
            r2_sub(b, c)) =
            r2_sub(r2_sub(a, b), r2_sub(b, c))
        r2_add(
            r2_add(
                r2_smul((Real.1 + Real.1 + Real.1), r2_add(r2_neg(r2_add(r2_sub(b, c), r2_neg(r2_add(r2_sub(a, b), r2_sub(b, c))))), r2_sub(b, c))),
                r2_smul(s, r2_rot90(r2_sub(r2_neg(r2_add(r2_sub(b, c), r2_neg(r2_add(r2_sub(a, b), r2_sub(b, c))))), r2_sub(b, c))))),
            r2_add(
                r2_smul((Real.1 + Real.1 + Real.1), r2_add(r2_neg(r2_add(r2_sub(b, c), r2_neg(r2_add(r2_sub(a, b), r2_sub(b, c))))), r2_sub(b, c))),
                r2_smul(s, r2_rot90(r2_sub(r2_neg(r2_add(r2_sub(b, c), r2_neg(r2_add(r2_sub(a, b), r2_sub(b, c))))), r2_sub(b, c)))))) =
            r2_add(
                r2_add(
                    r2_smul((Real.1 + Real.1 + Real.1), r2_add(r2_sub(a, b), r2_sub(b, c))),
                    r2_smul(s, r2_rot90(r2_sub(r2_sub(a, b), r2_sub(b, c))))),
                r2_add(
                    r2_smul((Real.1 + Real.1 + Real.1), r2_add(r2_sub(a, b), r2_sub(b, c))),
                    r2_smul(s, r2_rot90(r2_sub(r2_sub(a, b), r2_sub(b, c))))))
        r2_add(
            r2_smul((Real.1 + Real.1) * (Real.1 + Real.1 + Real.1), r2_sub(n1, n2)),
            r2_smul((Real.1 + Real.1) * (Real.1 + Real.1 + Real.1), r2_sub(n1, n2))) =
            napoleon_c(s, r2_sub(b, c), r2_neg(r2_add(r2_sub(a, b), r2_sub(b, c))))
        r2_neg(r2_add(
            r2_smul((Real.1 + Real.1) * (Real.1 + Real.1 + Real.1), r2_sub(n2, n3)),
            r2_smul(s, r2_rot90(r2_smul((Real.1 + Real.1) * (Real.1 + Real.1 + Real.1), r2_sub(n2, n3)))))) =
            r2_add(
                r2_smul((Real.1 + Real.1) * (Real.1 + Real.1 + Real.1), r2_sub(n1, n2)),
                r2_smul((Real.1 + Real.1) * (Real.1 + Real.1 + Real.1), r2_sub(n1, n2)))
        // |V2|^2 = |V1|^2
        napoleon_d120_norm(s, r2_smul((Real.1 + Real.1) * (Real.1 + Real.1 + Real.1), r2_sub(n2, n3)))
        r2_norm_sq(r2_neg(r2_add(
            r2_smul((Real.1 + Real.1) * (Real.1 + Real.1 + Real.1), r2_sub(n2, n3)),
            r2_smul(s, r2_rot90(r2_smul((Real.1 + Real.1) * (Real.1 + Real.1 + Real.1), r2_sub(n2, n3))))))) =
            (Real.1 + Real.1 + Real.1 + Real.1) *
            r2_norm_sq(r2_smul((Real.1 + Real.1) * (Real.1 + Real.1 + Real.1), r2_sub(n2, n3)))
        r2_norm_sq_double(r2_smul((Real.1 + Real.1) * (Real.1 + Real.1 + Real.1), r2_sub(n1, n2)))
        r2_norm_sq(r2_add(
            r2_smul((Real.1 + Real.1) * (Real.1 + Real.1 + Real.1), r2_sub(n1, n2)),
            r2_smul((Real.1 + Real.1) * (Real.1 + Real.1 + Real.1), r2_sub(n1, n2)))) =
            (Real.1 + Real.1) * ((Real.1 + Real.1) *
            r2_norm_sq(r2_smul((Real.1 + Real.1) * (Real.1 + Real.1 + Real.1), r2_sub(n1, n2))))
        (Real.1 + Real.1 + Real.1 + Real.1) *
            r2_norm_sq(r2_smul((Real.1 + Real.1) * (Real.1 + Real.1 + Real.1), r2_sub(n2, n3))) =
            (Real.1 + Real.1) * ((Real.1 + Real.1) *
            r2_norm_sq(r2_smul((Real.1 + Real.1) * (Real.1 + Real.1 + Real.1), r2_sub(n1, n2))))
        (Real.1 + Real.1 + Real.1 + Real.1) *
            r2_norm_sq(r2_smul((Real.1 + Real.1) * (Real.1 + Real.1 + Real.1), r2_sub(n2, n3))) =
            ((Real.1 + Real.1) + (Real.1 + Real.1)) *
            r2_norm_sq(r2_smul((Real.1 + Real.1) * (Real.1 + Real.1 + Real.1), r2_sub(n2, n3)))
        ((Real.1 + Real.1) + (Real.1 + Real.1)) *
            r2_norm_sq(r2_smul((Real.1 + Real.1) * (Real.1 + Real.1 + Real.1), r2_sub(n2, n3))) =
            (Real.1 + Real.1) * r2_norm_sq(r2_smul((Real.1 + Real.1) * (Real.1 + Real.1 + Real.1), r2_sub(n2, n3))) +
            (Real.1 + Real.1) * r2_norm_sq(r2_smul((Real.1 + Real.1) * (Real.1 + Real.1 + Real.1), r2_sub(n2, n3)))
        (Real.1 + Real.1) * r2_norm_sq(r2_smul((Real.1 + Real.1) * (Real.1 + Real.1 + Real.1), r2_sub(n2, n3))) +
            (Real.1 + Real.1) * r2_norm_sq(r2_smul((Real.1 + Real.1) * (Real.1 + Real.1 + Real.1), r2_sub(n2, n3))) =
            (Real.1 + Real.1) * ((Real.1 + Real.1) *
            r2_norm_sq(r2_smul((Real.1 + Real.1) * (Real.1 + Real.1 + Real.1), r2_sub(n2, n3))))
        (Real.1 + Real.1) * ((Real.1 + Real.1) *
            r2_norm_sq(r2_smul((Real.1 + Real.1) * (Real.1 + Real.1 + Real.1), r2_sub(n2, n3)))) =
            (Real.1 + Real.1) * ((Real.1 + Real.1) *
            r2_norm_sq(r2_smul((Real.1 + Real.1) * (Real.1 + Real.1 + Real.1), r2_sub(n1, n2))))
        napoleon_cancel4(
            r2_norm_sq(r2_smul((Real.1 + Real.1) * (Real.1 + Real.1 + Real.1), r2_sub(n2, n3))),
            r2_norm_sq(r2_smul((Real.1 + Real.1) * (Real.1 + Real.1 + Real.1), r2_sub(n1, n2))))
        r2_norm_sq(r2_smul((Real.1 + Real.1) * (Real.1 + Real.1 + Real.1), r2_sub(n2, n3))) =
            r2_norm_sq(r2_smul((Real.1 + Real.1) * (Real.1 + Real.1 + Real.1), r2_sub(n1, n2)))
        r2_norm_sq_smul((Real.1 + Real.1) * (Real.1 + Real.1 + Real.1), r2_sub(n2, n3))
        r2_norm_sq(r2_smul((Real.1 + Real.1) * (Real.1 + Real.1 + Real.1), r2_sub(n2, n3))) =
            (Real.1 + Real.1) * (Real.1 + Real.1 + Real.1) *
            ((Real.1 + Real.1) * (Real.1 + Real.1 + Real.1) * r2_norm_sq(r2_sub(n2, n3)))
        (Real.1 + Real.1) * (Real.1 + Real.1 + Real.1) *
            ((Real.1 + Real.1) * (Real.1 + Real.1 + Real.1) * r2_norm_sq(r2_sub(n2, n3))) =
            (Real.1 + Real.1) * (Real.1 + Real.1 + Real.1) *
            ((Real.1 + Real.1) * (Real.1 + Real.1 + Real.1) * r2_norm_sq(r2_sub(n1, n2)))
        napoleon_cancel(r2_norm_sq(r2_sub(n2, n3)), r2_norm_sq(r2_sub(n1, n2)))
        r2_norm_sq(r2_sub(n2, n3)) = r2_norm_sq(r2_sub(n1, n2))
        r2_norm_sq(r2_sub(n1, n2)) = r2_norm_sq(r2_sub(n2, n3)) and
            r2_norm_sq(r2_sub(n2, n3)) = r2_norm_sq(r2_sub(n3, n1))
    }
}
