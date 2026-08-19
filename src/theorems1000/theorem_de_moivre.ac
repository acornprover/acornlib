from nat import Nat, from_nat
from real import Real
from complex import Complex, complex_cos, complex_sin, complex_euler_formula,
    complex_exp, complex_i_mul_real, complex_exp_pow, complex_i_mul_real_mul

// De Moivre's theorem: for every natural number `n` and real `x`,
//     (Real.cos x + i Real.sin x)^n = (n x).cos + i (n x).sin.
// Abraham de Moivre discovered the formula around 1707; it connects
// trigonometry with the geometry of the complex plane and is the basis of
// Euler's formula.  The trigonometric functions here are the complex ones
// `complex_cos` and `complex_sin` applied to the real number `x` viewed as a
// complex number, and `Complex.i` is the imaginary unit; the power on the
// left is complex exponentiation.
//
// Proof (via Euler's formula): `complex_euler_formula` in the library proves
//     exp(i x) = Real.cos x + i Real.sin x,
// and `complex_exp_pow` proves the functional equation
//     exp(z)^n = exp(n z).
// Raising Euler's formula to the `n`-th power and applying the functional
// equation at `z = i x` gives
//     (Real.cos x + i Real.sin x)^n = exp(i x)^n = exp(i n x) = (n x).cos + i (n x).sin,
// where the last equality is Euler's formula again at `n x`.  The only
// bookkeeping step is moving the real scalar `n` past `i`, which is the
// library theorem `complex_i_mul_real_mul`.

/// De Moivre's theorem: (Real.cos x + i Real.sin x)^n = (n x).cos + i (n x).sin.
theorem theorems1000_de_moivre(n: Nat, x: Real) {
    (complex_cos(Complex.from_real(x)) + Complex.i * complex_sin(Complex.from_real(x))).pow(n) =
        complex_cos(Complex.from_real(from_nat[Real](n) * x)) +
            Complex.i * complex_sin(Complex.from_real(from_nat[Real](n) * x))
} by {
    // Euler's formula at x: exp(i x) = Real.cos x + i Real.sin x.
    complex_euler_formula(x)
    complex_exp(Complex.i * Complex.from_real(x)) =
        complex_cos(Complex.from_real(x)) + Complex.i * complex_sin(Complex.from_real(x))
    // Functional equation: exp(z)^n = exp(n z) at z = i x.
    complex_exp_pow(Complex.i * Complex.from_real(x), n)
    complex_exp(Complex.i * Complex.from_real(x)).pow(n) =
        complex_exp(Complex.from_real(from_nat[Real](n)) * (Complex.i * Complex.from_real(x)))
    // Move the real scalar n past i: n * (i x) = i (n x).
    complex_i_mul_real_mul(from_nat[Real](n), x)
    Complex.from_real(from_nat[Real](n)) * complex_i_mul_real(x) =
        complex_i_mul_real(from_nat[Real](n) * x)
    complex_i_mul_real(x) = Complex.i * Complex.from_real(x)
    complex_i_mul_real(from_nat[Real](n) * x) = Complex.i * Complex.from_real(from_nat[Real](n) * x)
    Complex.from_real(from_nat[Real](n)) * (Complex.i * Complex.from_real(x)) =
        Complex.i * Complex.from_real(from_nat[Real](n) * x)
    complex_exp(Complex.i * Complex.from_real(x)).pow(n) =
        complex_exp(Complex.i * Complex.from_real(from_nat[Real](n) * x))
    // Euler's formula at n x: exp(i (n x)) = (n x).cos + i (n x).sin.
    complex_euler_formula(from_nat[Real](n) * x)
    complex_exp(Complex.i * Complex.from_real(from_nat[Real](n) * x)) =
        complex_cos(Complex.from_real(from_nat[Real](n) * x)) +
            Complex.i * complex_sin(Complex.from_real(from_nat[Real](n) * x))
    // Assemble: (Real.cos x + i Real.sin x)^n = exp(i x)^n = ... = (n x).cos + i (n x).sin.
    (complex_cos(Complex.from_real(x)) + Complex.i * complex_sin(Complex.from_real(x))).pow(n) =
        complex_cos(Complex.from_real(from_nat[Real](n) * x)) +
            Complex.i * complex_sin(Complex.from_real(from_nat[Real](n) * x))
}
