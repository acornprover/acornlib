from pair import Pair
from real import Real
from top100 import r2_sub
from theorems1000.r2_helpers import r2_cross, r2_cross_sub_left, r2_cross_sub_right,
    r2_cross_self_zero, r2_cross_swap, r2_sub_sub_same_base

// Viviani's theorem: in an equilateral triangle, the sum of the distances
// from an interior point to the three sides equals the altitude.
//
// The library works with squared distances and cross products rather than
// with square roots, so the theorem is formalized in its area form, which is
// the standard proof of the classical statement: for any point `p` (interior
// or not) and any triangle `abc`, the sum of the (signed) double areas of
// the three triangles `pbc`, `pca` and `pab` equals the (signed) double area
// of `abc`:
//     cross(b - p, c - p) + cross(c - p, a - p) + cross(a - p, b - p)
//         = cross(b - a, c - a).
// When `abc` is equilateral and `p` lies inside it, each of the three
// triangles has the same base (the common side length) and its height is the
// distance from `p` to the corresponding side, so dividing both sides by the
// common side length gives the classical statement: the sum of the three
// distances equals the altitude of the equilateral triangle.
//
// The proof expands both sides with the bilinearity of the cross product and
// cancels the point-dependent terms pairwise.

/// Viviani's theorem, area form: the signed double areas of the three
/// subtriangles with apex `p` sum to the signed double area of `abc`.
theorem theorems1000_viviani_area(
    a: Pair[Real, Real], b: Pair[Real, Real], c: Pair[Real, Real],
    p: Pair[Real, Real]
) {
    r2_cross(r2_sub(b, p), r2_sub(c, p)) +
        r2_cross(r2_sub(c, p), r2_sub(a, p)) +
        r2_cross(r2_sub(a, p), r2_sub(b, p)) =
        r2_cross(r2_sub(b, a), r2_sub(c, a))
} by {
    // b - a = (b - p) - (a - p) and c - a = (c - p) - (a - p)
    r2_sub_sub_same_base(p, a, b)
    r2_sub(r2_sub(b, p), r2_sub(a, p)) = r2_sub(b, a)
    r2_sub_sub_same_base(p, a, c)
    r2_sub(r2_sub(c, p), r2_sub(a, p)) = r2_sub(c, a)
    // expand cross(b - a, c - a) = cross((b-p)-(a-p), (c-p)-(a-p))
    r2_cross_sub_left(r2_sub(b, p), r2_sub(a, p), r2_sub(r2_sub(c, p), r2_sub(a, p)))
    r2_cross(r2_sub(r2_sub(b, p), r2_sub(a, p)), r2_sub(r2_sub(c, p), r2_sub(a, p))) =
        r2_cross(r2_sub(b, p), r2_sub(r2_sub(c, p), r2_sub(a, p))) -
        r2_cross(r2_sub(a, p), r2_sub(r2_sub(c, p), r2_sub(a, p)))
    r2_cross(r2_sub(b, a), r2_sub(c, a)) =
        r2_cross(r2_sub(b, p), r2_sub(r2_sub(c, p), r2_sub(a, p))) -
        r2_cross(r2_sub(a, p), r2_sub(r2_sub(c, p), r2_sub(a, p)))
    // cross(b-p, (c-p)-(a-p)) = cross(b-p, c-p) - cross(b-p, a-p)
    r2_cross_sub_right(r2_sub(b, p), r2_sub(c, p), r2_sub(a, p))
    r2_cross(r2_sub(b, p), r2_sub(r2_sub(c, p), r2_sub(a, p))) =
        r2_cross(r2_sub(b, p), r2_sub(c, p)) - r2_cross(r2_sub(b, p), r2_sub(a, p))
    // cross(a-p, (c-p)-(a-p)) = cross(a-p, c-p) - cross(a-p, a-p) = cross(a-p, c-p)
    r2_cross_sub_right(r2_sub(a, p), r2_sub(c, p), r2_sub(a, p))
    r2_cross(r2_sub(a, p), r2_sub(r2_sub(c, p), r2_sub(a, p))) =
        r2_cross(r2_sub(a, p), r2_sub(c, p)) - r2_cross(r2_sub(a, p), r2_sub(a, p))
    r2_cross_self_zero(r2_sub(a, p))
    r2_cross(r2_sub(a, p), r2_sub(a, p)) = Real.0
    r2_cross(r2_sub(a, p), r2_sub(c, p)) - r2_cross(r2_sub(a, p), r2_sub(a, p)) =
        r2_cross(r2_sub(a, p), r2_sub(c, p)) - Real.0
    r2_cross(r2_sub(a, p), r2_sub(c, p)) - Real.0 =
        r2_cross(r2_sub(a, p), r2_sub(c, p))
    r2_cross(r2_sub(a, p), r2_sub(r2_sub(c, p), r2_sub(a, p))) =
        r2_cross(r2_sub(a, p), r2_sub(c, p))
    // cross(b-a, c-a) = cross(bp,cp) - cross(bp,ap) - cross(ap,cp)
    r2_cross(r2_sub(b, a), r2_sub(c, a)) =
        (r2_cross(r2_sub(b, p), r2_sub(c, p)) - r2_cross(r2_sub(b, p), r2_sub(a, p))) -
        r2_cross(r2_sub(a, p), r2_sub(c, p))
    // cross(bp,ap) = -cross(ap,bp) and cross(ap,cp) = -cross(cp,ap)
    r2_cross_swap(r2_sub(b, p), r2_sub(a, p))
    r2_cross(r2_sub(b, p), r2_sub(a, p)) = -r2_cross(r2_sub(a, p), r2_sub(b, p))
    r2_cross_swap(r2_sub(a, p), r2_sub(c, p))
    r2_cross(r2_sub(a, p), r2_sub(c, p)) = -r2_cross(r2_sub(c, p), r2_sub(a, p))
    // final reassembly: (X - (-Y)) - (-Z) = X + Y + Z with the goal's order
    r2_cross(r2_sub(b, a), r2_sub(c, a)) =
        (r2_cross(r2_sub(b, p), r2_sub(c, p)) - -r2_cross(r2_sub(a, p), r2_sub(b, p))) -
        -r2_cross(r2_sub(c, p), r2_sub(a, p))
    r2_cross(r2_sub(b, p), r2_sub(c, p)) - -r2_cross(r2_sub(a, p), r2_sub(b, p)) =
        r2_cross(r2_sub(b, p), r2_sub(c, p)) + r2_cross(r2_sub(a, p), r2_sub(b, p))
    - -r2_cross(r2_sub(c, p), r2_sub(a, p)) = r2_cross(r2_sub(c, p), r2_sub(a, p))
    (r2_cross(r2_sub(b, p), r2_sub(c, p)) + r2_cross(r2_sub(a, p), r2_sub(b, p))) -
        -r2_cross(r2_sub(c, p), r2_sub(a, p)) =
        (r2_cross(r2_sub(b, p), r2_sub(c, p)) + r2_cross(r2_sub(a, p), r2_sub(b, p))) +
        -(-r2_cross(r2_sub(c, p), r2_sub(a, p)))
    (r2_cross(r2_sub(b, p), r2_sub(c, p)) + r2_cross(r2_sub(a, p), r2_sub(b, p))) +
        -(-r2_cross(r2_sub(c, p), r2_sub(a, p))) =
        (r2_cross(r2_sub(b, p), r2_sub(c, p)) + r2_cross(r2_sub(a, p), r2_sub(b, p))) +
        r2_cross(r2_sub(c, p), r2_sub(a, p))
    (r2_cross(r2_sub(b, p), r2_sub(c, p)) + r2_cross(r2_sub(a, p), r2_sub(b, p))) +
        r2_cross(r2_sub(c, p), r2_sub(a, p)) =
        r2_cross(r2_sub(b, p), r2_sub(c, p)) + r2_cross(r2_sub(a, p), r2_sub(b, p)) +
        r2_cross(r2_sub(c, p), r2_sub(a, p))
    r2_cross(r2_sub(b, a), r2_sub(c, a)) =
        r2_cross(r2_sub(b, p), r2_sub(c, p)) + r2_cross(r2_sub(a, p), r2_sub(b, p)) +
        r2_cross(r2_sub(c, p), r2_sub(a, p))
    r2_cross(r2_sub(b, a), r2_sub(c, a)) =
        r2_cross(r2_sub(b, p), r2_sub(c, p)) + r2_cross(r2_sub(c, p), r2_sub(a, p)) +
        r2_cross(r2_sub(a, p), r2_sub(b, p))
}
