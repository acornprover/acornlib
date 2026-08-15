from pair import Pair
from real import Real
from top100 import r2_add, r2_neg, r2_sub, r2_dot, r2_norm_sq, r2_orthogonal,
    top100_004_pythagorean_theorem
from theorems1000.r2_helpers import r2_sub_through, r2_sub_eq_add_neg,
    r2_sub_reverse_neg, r2_orthogonal_neg_left, r2_norm_sq_add_expansion,
    r2_norm_sq_sub_expansion, r2_dot_comm, r2_add_zero_left, r2_add_neg_right,
    r2_sub_add_pair_distrib, r2_add_sub_left_cancel, r2_sub_sub_swap, r2_add_zero_right
from algebra.field.field import field_mul_eq_zero
from algebra.add_comm_group import sub_eq_zero_imp_eq
from algebra.add_group import inverse_left, right_cancel
from algebra.add_ordered_group import negative_of_nonnegative
from order import lte_antisymm

numerals Real

// Ptolemy's theorem: in a cyclic quadrilateral, the product of the diagonals
// equals the sum of the products of the opposite sides.  For the cyclic
// quadrilateral `abcd` with diagonals `ac` and `bd`, the identity reads
//     AC * BD = AB * CD + BC * AD.
// As in the library's Heron and intersecting-chords theorems, the side and
// diagonal lengths are given as witnesses: `w^2 = |b-a|^2`, `x^2 = |c-b|^2`,
// `y^2 = |d-c|^2`, `z^2 = |a-d|^2` for the sides and `p^2 = |c-a|^2`,
// `q^2 = |d-b|^2` for the diagonals, so the conclusion is the real identity
//     p * q = w * y + x * z.
//
// The general statement (for the four vertices on a common circle) is
// recorded at the bottom of this file.  The special case of a rectangle is
// proved below as `theorems1000_ptolemy_rectangle`: a rectangle is a cyclic
// quadrilateral, and the identity collapses to `AC^2 = AB^2 + BC^2`
// (Pythagoras) because the opposite sides and the diagonals of a rectangle
// are equal.  The library proves the same statement as `ptolemy_rectangle`
// in `geometry/ptolemy.ac` (for a rectangle given by its right angle and
// parallelogram structure); the proof below develops it from the public
// `top100` Pythagorean theorem and the `r2` vector algebra of this
// directory.
//
// The rectangle hypotheses are the two structural facts used by the
// library's `is_rectangle` predicate: the parallelogram condition
// `a + c = b + d` (the diagonals share a midpoint) and the right angle at
// `b`, `(a - b) . (c - b) = 0`.

/// The difference of squares factorisation over the reals.
theorem ptolemy_diff_of_squares(x: Real, y: Real) {
    (x - y) * (x + y) = x * x - y * y
} by {
    (x - y) * (x + y) = (x - y) * x + (x - y) * y
    (x - y) * x = x * x - y * x
    (x - y) * y = x * y - y * y
    (x - y) * (x + y) = (x * x - y * x) + (x * y - y * y)
    y * x = x * y
    x * x - y * x + (x * y - y * y) = x * x - x * y + (x * y - y * y)
    x * x - x * y + (x * y - y * y) = x * x - y * y
}

/// Equal squares with nonnegative witnesses are equal: `x >= 0`, `y >= 0`
/// and `x^2 = y^2` give `x = y`.
theorem ptolemy_nonneg_sq_eq(x: Real, y: Real) {
    Real.0 <= x and Real.0 <= y and x * x = y * y implies x = y
} by {
    if Real.0 <= x and Real.0 <= y and x * x = y * y {
        Real.0 <= x
        Real.0 <= y
        x * x = y * y
        ptolemy_diff_of_squares(x, y)
        (x - y) * (x + y) = x * x - y * y
        x * x - y * y = Real.0
        (x - y) * (x + y) = Real.0
        field_mul_eq_zero(x - y, x + y)
        x - y = Real.0 or x + y = Real.0
        if x - y = Real.0 {
            sub_eq_zero_imp_eq(x, y)
            x = y
        } else {
            x + y = Real.0
            // x + y = 0 with x, y >= 0 forces x = 0 and y = 0
            inverse_left(y)
            -y + y = Real.0
            x + y = -y + y
            right_cancel(x, -y, y)
            x = -y
            Real.0 <= -y
            negative_of_nonnegative(-y)
            -(-y) <= Real.0
            -(-y) = y
            y <= Real.0
            lte_antisymm(y, Real.0)
            y = Real.0
            x = Real.0 - y
            x = Real.0
            x = y
        }
    }
}

/// Subtracting the same vector from equal vectors gives equal results.
theorem ptolemy_sub_cong_left(x: Pair[Real, Real], y: Pair[Real, Real], z: Pair[Real, Real]) {
    x = y implies r2_sub(x, z) = r2_sub(y, z)
} by {
    if x = y {
        r2_sub(x, z) = r2_sub(y, z)
    }
}

/// Subtracting a vanishing term twice leaves a sum unchanged:
/// `d = 0` gives `x - d - d = x`.
theorem ptolemy_sub_zero_twice(x: Real, d: Real) {
    d = Real.0 implies x - d - d = x
} by {
    if d = Real.0 {
        x - d - d = x - Real.0 - Real.0
        x - Real.0 - Real.0 = x
    }
}

/// Adding a vanishing term twice leaves a sum unchanged:
/// `d = 0` gives `x + d + d = x`.
theorem ptolemy_add_zero_twice(x: Real, d: Real) {
    d = Real.0 implies x + d + d = x
} by {
    if d = Real.0 {
        x + d + d = x + Real.0 + Real.0
        x + Real.0 + Real.0 = x
    }
}

/// A point minus itself is the origin.
theorem ptolemy_sub_self(a: Pair[Real, Real]) {
    r2_sub(a, a) = Pair.new(Real.0, Real.0)
} by {
    r2_sub_eq_add_neg(a, a)
    r2_sub(a, a) = r2_add(a, r2_neg(a))
    r2_add_neg_right(a)
    r2_add(a, r2_neg(a)) = Pair.new(Real.0, Real.0)
    r2_sub(a, a) = Pair.new(Real.0, Real.0)
}

/// In a parallelogram the vector `d - c` equals `a - b`.
theorem ptolemy_parall_dc(a: Pair[Real, Real], b: Pair[Real, Real], c: Pair[Real, Real], d: Pair[Real, Real]) {
    r2_add(a, c) = r2_add(b, d) implies r2_sub(d, c) = r2_sub(a, b)
} by {
    if r2_add(a, c) = r2_add(b, d) {
        // (a + c) - (b + c) = (b + d) - (b + c)
        ptolemy_sub_cong_left(r2_add(a, c), r2_add(b, d), r2_add(b, c))
        r2_sub(r2_add(a, c), r2_add(b, c)) = r2_sub(r2_add(b, d), r2_add(b, c))
        // (b + d) - (b + c) = (b - b) + (d - c) = d - c
        r2_sub_add_pair_distrib(b, d, b, c)
        r2_sub(r2_add(b, d), r2_add(b, c)) = r2_add(r2_sub(b, b), r2_sub(d, c))
        ptolemy_sub_self(b)
        r2_sub(b, b) = Pair.new(Real.0, Real.0)
        r2_add_zero_left(r2_sub(d, c))
        r2_add(Pair.new(Real.0, Real.0), r2_sub(d, c)) = r2_sub(d, c)
        r2_add(r2_sub(b, b), r2_sub(d, c)) = r2_sub(d, c)
        r2_sub(r2_add(b, d), r2_add(b, c)) = r2_sub(d, c)
        // (a + c) - (b + c) = (a - b) + (c - c) = a - b
        r2_sub_add_pair_distrib(a, c, b, c)
        r2_sub(r2_add(a, c), r2_add(b, c)) = r2_add(r2_sub(a, b), r2_sub(c, c))
        ptolemy_sub_self(c)
        r2_sub(c, c) = Pair.new(Real.0, Real.0)
        r2_add_zero_right(r2_sub(a, b))
        r2_add(r2_sub(a, b), Pair.new(Real.0, Real.0)) = r2_sub(a, b)
        r2_add(r2_sub(a, b), r2_sub(c, c)) = r2_sub(a, b)
        r2_sub(r2_add(a, c), r2_add(b, c)) = r2_sub(a, b)
        // chain
        r2_sub(d, c) = r2_sub(a, b)
    }
}

/// In a parallelogram the vector `a - d` equals `b - c`.
theorem ptolemy_parall_ad(a: Pair[Real, Real], b: Pair[Real, Real], c: Pair[Real, Real], d: Pair[Real, Real]) {
    r2_add(a, c) = r2_add(b, d) implies r2_sub(a, d) = r2_sub(b, c)
} by {
    if r2_add(a, c) = r2_add(b, d) {
        // a = (b + d) - c
        ptolemy_sub_cong_left(r2_add(a, c), r2_add(b, d), c)
        r2_sub(r2_add(a, c), c) = r2_sub(r2_add(b, d), c)
        r2_add_sub_left_cancel(a, c)
        r2_sub(r2_add(a, c), c) = a
        a = r2_sub(r2_add(b, d), c)
        // a - d = (b + d) - c - d = (b + d) - d - c = b - c
        r2_sub(a, d) = r2_sub(r2_sub(r2_add(b, d), c), d)
        r2_sub_sub_swap(r2_add(b, d), c, d)
        r2_sub(r2_sub(r2_add(b, d), c), d) = r2_sub(r2_sub(r2_add(b, d), d), c)
        r2_add_sub_left_cancel(b, d)
        r2_sub(r2_add(b, d), d) = b
        r2_sub(r2_sub(r2_add(b, d), d), c) = r2_sub(b, c)
        r2_sub(a, d) = r2_sub(b, c)
    }
}

/// The squared diagonals of a right-angled parallelogram are equal.
theorem ptolemy_diag_norm_eq(a: Pair[Real, Real], b: Pair[Real, Real], c: Pair[Real, Real], d: Pair[Real, Real]) {
    r2_add(a, c) = r2_add(b, d) and r2_orthogonal(r2_sub(a, b), r2_sub(c, b)) implies
    r2_norm_sq(r2_sub(c, a)) = r2_norm_sq(r2_sub(d, b))
} by {
    if r2_add(a, c) = r2_add(b, d) and r2_orthogonal(r2_sub(a, b), r2_sub(c, b)) {
        // u = a - b, v = c - b; the orthogonality is u . v = 0
        r2_orthogonal(r2_sub(a, b), r2_sub(c, b))
        r2_dot(r2_sub(a, b), r2_sub(c, b)) = Real.0
        r2_dot_comm(r2_sub(a, b), r2_sub(c, b))
        r2_dot(r2_sub(a, b), r2_sub(c, b)) = r2_dot(r2_sub(c, b), r2_sub(a, b))
        r2_dot(r2_sub(c, b), r2_sub(a, b)) = Real.0
        // c - a = v - u
        r2_sub_through(a, b, c)
        r2_sub(c, a) = r2_add(r2_sub(b, a), r2_sub(c, b))
        r2_sub_reverse_neg(b, a)
        r2_sub(b, a) = r2_neg(r2_sub(a, b))
        r2_sub(c, a) = r2_add(r2_neg(r2_sub(a, b)), r2_sub(c, b))
        r2_sub_eq_add_neg(r2_sub(c, b), r2_sub(a, b))
        r2_sub(r2_sub(c, b), r2_sub(a, b)) =
            r2_add(r2_sub(c, b), r2_neg(r2_sub(a, b)))
        r2_sub(c, a) = r2_sub(r2_sub(c, b), r2_sub(a, b))
        // d - b = v + u
        r2_sub_through(c, b, d)
        r2_sub(d, b) = r2_add(r2_sub(c, b), r2_sub(d, c))
        ptolemy_parall_dc(a, b, c, d)
        r2_sub(d, c) = r2_sub(a, b)
        r2_sub(d, b) = r2_add(r2_sub(c, b), r2_sub(a, b))
        // |v - u|^2 = |v|^2 + |u|^2 (u . v = 0)
        r2_norm_sq_sub_expansion(r2_sub(c, b), r2_sub(a, b))
        r2_norm_sq(r2_sub(r2_sub(c, b), r2_sub(a, b))) =
            r2_norm_sq(r2_sub(c, b)) + r2_norm_sq(r2_sub(a, b)) -
            r2_dot(r2_sub(c, b), r2_sub(a, b)) - r2_dot(r2_sub(c, b), r2_sub(a, b))
        ptolemy_sub_zero_twice(r2_norm_sq(r2_sub(c, b)) + r2_norm_sq(r2_sub(a, b)),
            r2_dot(r2_sub(c, b), r2_sub(a, b)))
        r2_norm_sq(r2_sub(c, b)) + r2_norm_sq(r2_sub(a, b)) -
            r2_dot(r2_sub(c, b), r2_sub(a, b)) - r2_dot(r2_sub(c, b), r2_sub(a, b)) =
            r2_norm_sq(r2_sub(c, b)) + r2_norm_sq(r2_sub(a, b))
        r2_norm_sq(r2_sub(r2_sub(c, b), r2_sub(a, b))) =
            r2_norm_sq(r2_sub(c, b)) + r2_norm_sq(r2_sub(a, b))
        // |v + u|^2 = |v|^2 + |u|^2 (u . v = 0)
        r2_norm_sq_add_expansion(r2_sub(c, b), r2_sub(a, b))
        r2_norm_sq(r2_add(r2_sub(c, b), r2_sub(a, b))) =
            r2_norm_sq(r2_sub(c, b)) + r2_norm_sq(r2_sub(a, b)) +
            r2_dot(r2_sub(c, b), r2_sub(a, b)) + r2_dot(r2_sub(c, b), r2_sub(a, b))
        ptolemy_add_zero_twice(r2_norm_sq(r2_sub(c, b)) + r2_norm_sq(r2_sub(a, b)),
            r2_dot(r2_sub(c, b), r2_sub(a, b)))
        r2_norm_sq(r2_sub(c, b)) + r2_norm_sq(r2_sub(a, b)) +
            r2_dot(r2_sub(c, b), r2_sub(a, b)) + r2_dot(r2_sub(c, b), r2_sub(a, b)) =
            r2_norm_sq(r2_sub(c, b)) + r2_norm_sq(r2_sub(a, b))
        r2_norm_sq(r2_add(r2_sub(c, b), r2_sub(a, b))) =
            r2_norm_sq(r2_sub(c, b)) + r2_norm_sq(r2_sub(a, b))
        r2_norm_sq(r2_sub(r2_sub(c, b), r2_sub(a, b))) =
            r2_norm_sq(r2_add(r2_sub(c, b), r2_sub(a, b)))
        r2_norm_sq(r2_sub(c, a)) = r2_norm_sq(r2_sub(d, b))
    }
}

/// Ptolemy's theorem for a rectangle: the product of the diagonals equals
/// the sum of the products of the opposite sides, i.e. Pythagoras.
///
/// With `a + c = b + d` (the parallelogram condition, so the diagonals share
/// a midpoint) and the right angle at `b` (`(a - b) . (c - b) = 0`), the
/// four sides and two diagonals of the rectangle satisfy
/// `w = y`, `x = z`, `p = q` and `p^2 = w^2 + x^2`; hence
/// `p q = p^2 = w^2 + x^2 = w y + x z`.
theorem theorems1000_ptolemy_rectangle(
    a: Pair[Real, Real], b: Pair[Real, Real], c: Pair[Real, Real], d: Pair[Real, Real],
    w: Real, x: Real, y: Real, z: Real, p: Real, q: Real
) {
    r2_add(a, c) = r2_add(b, d) and
    r2_orthogonal(r2_sub(a, b), r2_sub(c, b)) and
    w * w = r2_norm_sq(r2_sub(b, a)) and
    x * x = r2_norm_sq(r2_sub(c, b)) and
    y * y = r2_norm_sq(r2_sub(d, c)) and
    z * z = r2_norm_sq(r2_sub(a, d)) and
    p * p = r2_norm_sq(r2_sub(c, a)) and
    q * q = r2_norm_sq(r2_sub(d, b)) and
    Real.0 <= w and Real.0 <= x and Real.0 <= y and Real.0 <= z and
    Real.0 <= p and Real.0 <= q
    implies p * q = w * y + x * z
} by {
    if r2_add(a, c) = r2_add(b, d) and
        r2_orthogonal(r2_sub(a, b), r2_sub(c, b)) and
        w * w = r2_norm_sq(r2_sub(b, a)) and
        x * x = r2_norm_sq(r2_sub(c, b)) and
        y * y = r2_norm_sq(r2_sub(d, c)) and
        z * z = r2_norm_sq(r2_sub(a, d)) and
        p * p = r2_norm_sq(r2_sub(c, a)) and
        q * q = r2_norm_sq(r2_sub(d, b)) and
        Real.0 <= w and Real.0 <= x and Real.0 <= y and Real.0 <= z and
        Real.0 <= p and Real.0 <= q {
        // the opposite sides are equal: y = w and z = x
        ptolemy_parall_dc(a, b, c, d)
        r2_sub(d, c) = r2_sub(a, b)
        r2_norm_sq(r2_sub(d, c)) = r2_norm_sq(r2_sub(a, b))
        r2_norm_sq(r2_sub(a, b)) = r2_norm_sq(r2_sub(b, a))
        r2_norm_sq(r2_sub(d, c)) = r2_norm_sq(r2_sub(b, a))
        w * w = y * y
        ptolemy_nonneg_sq_eq(w, y)
        w = y
        ptolemy_parall_ad(a, b, c, d)
        r2_sub(a, d) = r2_sub(b, c)
        r2_norm_sq(r2_sub(a, d)) = r2_norm_sq(r2_sub(b, c))
        x * x = z * z
        ptolemy_nonneg_sq_eq(x, z)
        x = z
        // the diagonals are equal: p = q
        ptolemy_diag_norm_eq(a, b, c, d)
        r2_norm_sq(r2_sub(c, a)) = r2_norm_sq(r2_sub(d, b))
        p * p = q * q
        ptolemy_nonneg_sq_eq(p, q)
        p = q
        // Pythagoras at the right angle b: p^2 = w^2 + x^2
        top100_004_pythagorean_theorem(a, b, c)
        r2_norm_sq(r2_sub(c, a)) = r2_norm_sq(r2_sub(b, a)) + r2_norm_sq(r2_sub(c, b))
        p * p = w * w + x * x
        // p q = p^2 = w^2 + x^2 = w y + x z
        p * q = p * p
        p * q = w * w + x * x
        w * y = w * w
        x * z = x * x
        w * w + x * x = w * y + x * z
        p * q = w * y + x * z
    }
}

// The general statement for a cyclic quadrilateral is recorded without
// proof; it needs the inscribed-angle machinery, which is not yet public.
//
// theorem theorems1000_ptolemy(
//     center: Pair[Real, Real], radius_sq: Real,
//     a: Pair[Real, Real], b: Pair[Real, Real],
//     c: Pair[Real, Real], d: Pair[Real, Real],
//     w: Real, x: Real, y: Real, z: Real, p: Real, q: Real
// ) {
//     r2_norm_sq(r2_sub(a, center)) = radius_sq and
//     r2_norm_sq(r2_sub(b, center)) = radius_sq and
//     r2_norm_sq(r2_sub(c, center)) = radius_sq and
//     r2_norm_sq(r2_sub(d, center)) = radius_sq and
//     w * w = r2_norm_sq(r2_sub(b, a)) and
//     x * x = r2_norm_sq(r2_sub(c, b)) and
//     y * y = r2_norm_sq(r2_sub(d, c)) and
//     z * z = r2_norm_sq(r2_sub(a, d)) and
//     p * p = r2_norm_sq(r2_sub(c, a)) and
//     q * q = r2_norm_sq(r2_sub(d, b))
//     implies
//     p * q = w * y + x * z
// }
