from pair import Pair
from real import Real
from top100 import r2_add, r2_sub
from theorems1000.r2_helpers import r2_parallelogram, r2_parallelogram_apply,
    r2_double_pair_rearrange, r2_double_eq_double, r2_add_four_rearrange

// Varignon's theorem: the midpoints of the sides of any quadrilateral form a
// parallelogram.  Here `m1`, `m2`, `m3` and `m4` are the midpoints of the
// sides `ab`, `bc`, `cd` and `da`, written as the midpoint hypotheses
// `m1 + m1 = a + b` and so on, and the parallelogram condition for the
// midpoint quadrilateral is that its diagonals share a midpoint:
// `m1 + m3 = m2 + m4`.

/// Varignon's theorem: the side midpoints of a quadrilateral form a
/// parallelogram.
///
/// With `m1`, `m2`, `m3`, `m4` the midpoints of `ab`, `bc`, `cd`, `da`, the
/// quadrilateral `m1 m2 m3 m4` is a parallelogram: its diagonals `m1 m3` and
/// `m2 m4` share a midpoint, i.e. `m1 + m3 = m2 + m4`.
theorem theorems1000_varignon(
    a: Pair[Real, Real], b: Pair[Real, Real], c: Pair[Real, Real], d: Pair[Real, Real],
    m1: Pair[Real, Real], m2: Pair[Real, Real], m3: Pair[Real, Real], m4: Pair[Real, Real]
) {
    r2_add(m1, m1) = r2_add(a, b) and
    r2_add(m2, m2) = r2_add(b, c) and
    r2_add(m3, m3) = r2_add(c, d) and
    r2_add(m4, m4) = r2_add(d, a)
    implies r2_parallelogram(m1, m2, m3, m4)
} by {
    if r2_add(m1, m1) = r2_add(a, b) and
        r2_add(m2, m2) = r2_add(b, c) and
        r2_add(m3, m3) = r2_add(c, d) and
        r2_add(m4, m4) = r2_add(d, a) {
        r2_add(m1, m1) = r2_add(a, b)
        r2_add(m2, m2) = r2_add(b, c)
        r2_add(m3, m3) = r2_add(c, d)
        r2_add(m4, m4) = r2_add(d, a)
        // 2(m1 + m3) = 2m1 + 2m3 = (a + b) + (c + d)
        r2_double_pair_rearrange(m1, m3)
        r2_add(r2_add(m1, m1), r2_add(m3, m3)) = r2_add(r2_add(m1, m3), r2_add(m1, m3))
        r2_add(r2_add(m1, m1), r2_add(m3, m3)) = r2_add(r2_add(a, b), r2_add(c, d))
        r2_add(r2_add(m1, m3), r2_add(m1, m3)) = r2_add(r2_add(a, b), r2_add(c, d))
        // 2(m2 + m4) = 2m2 + 2m4 = (b + c) + (d + a)
        r2_double_pair_rearrange(m2, m4)
        r2_add(r2_add(m2, m2), r2_add(m4, m4)) = r2_add(r2_add(m2, m4), r2_add(m2, m4))
        r2_add(r2_add(m2, m2), r2_add(m4, m4)) = r2_add(r2_add(b, c), r2_add(d, a))
        r2_add(r2_add(m2, m4), r2_add(m2, m4)) = r2_add(r2_add(b, c), r2_add(d, a))
        // (a + b) + (c + d) = (b + c) + (d + a)
        r2_add_four_rearrange(a, b, c, d)
        r2_add(r2_add(a, b), r2_add(c, d)) = r2_add(r2_add(b, c), r2_add(d, a))
        r2_add(r2_add(m1, m3), r2_add(m1, m3)) = r2_add(r2_add(m2, m4), r2_add(m2, m4))
        // cancel the doubling
        r2_double_eq_double(r2_add(m1, m3), r2_add(m2, m4))
        r2_add(m1, m3) = r2_add(m2, m4)
        r2_parallelogram_apply(m1, m2, m3, m4)
        r2_parallelogram(m1, m2, m3, m4) = (r2_add(m1, m3) = r2_add(m2, m4))
        r2_parallelogram(m1, m2, m3, m4)
    }
}
