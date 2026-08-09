from pair import Pair
from real import Real
from top100 import r2_add, r2_sub
from theorems1000.r2_helpers import r2_smul, r2_cross

// Pappus's hexagon theorem: in the affine plane, if the points `a`, `b`, `c`
// lie on one line and the points `a'`, `b'`, `c'` lie on another line, then
// the three intersection points
//     x = ab' ∩ a'b,   y = ac' ∩ a'c,   z = bc' ∩ b'c
// are collinear.  This is the first nontrivial theorem of projective
// geometry (it holds over any field, with the two lines possibly parallel or
// meeting).
//
// The statement below is entirely elementary.  The two lines are given by a
// base point and a direction: `b = a + s1 u`, `c = a + s2 u` on the first
// line and `b' = a' + t1 v`, `c' = a' + t2 v` on the second.  Each
// intersection point is witnessed by its two line parameters, e.g.
// `x = a + p1 (b' - a) = a' + q1 (b - a')`; the conclusion is the
// collinearity of `x`, `y`, `z` in the form `cross(y - x, z - x) = 0`.
//
// Proof sketch: choose coordinates with the two lines as the coordinate
// axes.  (Any two distinct directions `u`, `v` can be mapped to the axes by
// an invertible affine map, which preserves incidence and collinearity.)
// Writing `a = (0, 0)`, `b = (1, 0)`, `c = (s, 0)` on the x-axis and
// `a' = (0, 1)`, `b' = (1, 1)`, `c' = (t, 1)` on the parallel line
// `y = 1`, the three intersections have rational coordinates:
//     x = ((s - t)/(s - 1), s (t - 1)/(s - 1)), ...
// (explicit formulas are obtained by solving the two line equations), and a
// direct cross-product computation gives `cross(y - x, z - x) = 0` after
// clearing denominators — a polynomial identity in `s` and `t`.  The
// prover-visible version of this proof would develop the two-by-two linear
// solvers for the intersection parameters and then expand the final cross
// product with the bilinearity lemmas (`r2_cross_add_left`,
// `r2_cross_smul_left`, `r2_cross_swap`, `r2_cross_self_zero`), as in the
// Menelaus and Ceva sketches; the statement is recorded here with the
// witness parameters so that the polynomial identity is explicit.
//
// theorem theorems1000_pappus_hexagon(
//     a1: Pair[Real, Real], b1: Pair[Real, Real], c1: Pair[Real, Real],
//     u: Pair[Real, Real],
//     a2: Pair[Real, Real], b2: Pair[Real, Real], c2: Pair[Real, Real],
//     v: Pair[Real, Real],
//     s1: Real, s2: Real, t1: Real, t2: Real,
//     x: Pair[Real, Real], y: Pair[Real, Real], z: Pair[Real, Real],
//     p1: Real, q1: Real, p2: Real, q2: Real, p3: Real, q3: Real
// ) {
//     r2_add(a1, r2_smul(s1, u)) = b1 and
//     r2_add(a1, r2_smul(s2, u)) = c1 and
//     r2_add(a2, r2_smul(t1, v)) = b2 and
//     r2_add(a2, r2_smul(t2, v)) = c2 and
//     r2_add(a1, r2_smul(p1, r2_sub(b2, a1))) = x and
//     r2_add(a2, r2_smul(q1, r2_sub(b1, a2))) = x and
//     r2_add(a1, r2_smul(p2, r2_sub(c2, a1))) = y and
//     r2_add(a2, r2_smul(q2, r2_sub(c1, a2))) = y and
//     r2_add(b1, r2_smul(p3, r2_sub(c2, b1))) = z and
//     r2_add(b2, r2_smul(q3, r2_sub(c1, b2))) = z
//     implies
//     r2_cross(r2_sub(y, x), r2_sub(z, x)) = Real.0
// }
