from nat import Nat

numerals Nat

// Dirichlet's theorem on arithmetic progressions (1837): every arithmetic
// progression
//     a, a + d, a + 2d, a + 3d, ...
// with `a` and `d` coprime contains infinitely many primes.  In symbols:
//     for every pair (a, d) with gcd(a, d) = 1 and every bound n, there is
//     a prime p > n with p ≡ a (mod d).
// The theorem was proved by Peter Gustav Lejeune Dirichlet using Dirichlet
// characters and the non-vanishing of the Dirichlet L-function L(s, chi) at
// s = 1; it is the reason the analytic theory of L-functions was created.
// The case d = 1 is Euclid's theorem on the infinitude of primes (proved in
// `src/top100/theorem_011_infinitude_of_primes.ac`); the case a = d = 2 is
// the infinitude of odd primes.
//
// The statement is recorded with the library's natural-number arithmetic:
// `p.is_prime` is primality, `p.congr_mod(a, d)` is congruence modulo `d`,
// and `a.coprime(d)` is gcd(a, d) = 1.  The "infinitely many" quantifier is
// expressed in the same form as the library's infinitude of primes: for
// every bound `n` there is a prime beyond it.
//
// Proof sketch: consider the Dirichlet series
//     sum_{p prime} chi(p) / p^s
// over primes in the progression, obtained from the Euler product of the
// L-function L(s, chi) = sum chi(n)/n^s; the theorem follows from the fact
// that L(1, chi) != 0 for every non-principal character chi mod d.  The
// proof needs complex analysis (analytic continuation of L-functions,
// non-vanishing of L(1, chi)) which is not in the library; the statement is
// therefore recorded but not proved.
//
// theorem theorems1000_dirichlet_ap(a: Nat, d: Nat) {
//     a.coprime(d) implies forall(n: Nat) {
//         exists(p: Nat) {
//             n < p and p.is_prime and p.congr_mod(a, d)
//         }
//     }
// }
