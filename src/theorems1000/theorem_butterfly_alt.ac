from pair import Pair
from real import Real
from top100 import r2_add, r2_sub, r2_norm_sq
from theorems1000.r2_helpers import r2_cross

// The butterfly theorem: let `pq` be a chord of a circle with midpoint `m`;
// let `ab` and `cd` be two further chords through `m`.  If the chords `ad`
// and `bc` meet `pq` at `x` and `y`, then `m` is the midpoint of `xy`:
// |x - m|^2 = |y - m|^2.
//
// The circle has centre `o` and squared radius `radius_sq` (the hypotheses
// `|a-o|^2 = ... = |q-o|^2 = radius_sq`), `m` is the midpoint of `pq`
// (`p + q = 2m`), and `m` lies on the two chords `ab` and `cd`
// (`cross(m - a, b - a) = 0`, `cross(m - c, d - c) = 0`).  The points `x`
// and `y` are witnessed by their collinearity with the two lines they lie
// on: `x` on `ad` and on `pq`, `y` on `bc` and on `pq`.
//
// The statement is commented out because the proof needs the power of the
// point `m` with respect to the circle (the intersecting-chords product
// `|a-m| * |b-m| = |c-m| * |d-m|`, available in squared form in the
// library's intersecting-chords theorem) together with a long expansion of
// the two intersections `x` and `y` with the diameter line `pq`; the sketch
// is recorded below.
//
// Proof sketch (coordinate form): place `m` at the origin and `pq` on the
// x-axis, so `p = (-t, 0)` and `q = (t, 0)` with `t = |p - m| = |q - m|`.
// A chord through the origin with direction `(u, v)` meets the circle
// `X^2 + Y^2 = R^2` at the two points `+-lambda (u, v)` with
// `lambda^2 (u^2 + v^2) = R^2`, so the four chord endpoints of the chords
// `ab` and `cd` are `a = alpha u`, `b = -alpha u`, `c = beta v`,
// `d = -beta v` for two vectors `u`, `v` (with the sign convention of the
// two chords through `m`).  The lines `ad` and `bc` are the joins of
// `alpha u` with `-beta v` and of `-alpha u` with `beta v`; each meets the
// x-axis at a point whose coordinate is proportional to `alpha beta`
// divided by the y-component of the join direction.  Expanding the two
// intersections shows they are symmetric about the origin, so `m` is the
// midpoint of `xy`.  The algebraic expansion is long but purely rational.
//
// theorem theorems1000_butterfly(
//     o: Pair[Real, Real], radius_sq: Real,
//     a: Pair[Real, Real], b: Pair[Real, Real], c: Pair[Real, Real], d: Pair[Real, Real],
//     p: Pair[Real, Real], q: Pair[Real, Real], m: Pair[Real, Real],
//     x: Pair[Real, Real], y: Pair[Real, Real]
// ) {
//     r2_norm_sq(r2_sub(a, o)) = radius_sq and
//     r2_norm_sq(r2_sub(b, o)) = radius_sq and
//     r2_norm_sq(r2_sub(c, o)) = radius_sq and
//     r2_norm_sq(r2_sub(d, o)) = radius_sq and
//     r2_norm_sq(r2_sub(p, o)) = radius_sq and
//     r2_norm_sq(r2_sub(q, o)) = radius_sq and
//     r2_add(p, q) = r2_add(m, m) and
//     r2_cross(r2_sub(m, a), r2_sub(b, a)) = Real.0 and
//     r2_cross(r2_sub(m, c), r2_sub(d, c)) = Real.0 and
//     r2_cross(r2_sub(x, a), r2_sub(d, a)) = Real.0 and
//     r2_cross(r2_sub(x, p), r2_sub(q, p)) = Real.0 and
//     r2_cross(r2_sub(y, b), r2_sub(c, b)) = Real.0 and
//     r2_cross(r2_sub(y, p), r2_sub(q, p)) = Real.0
//     implies
//     r2_norm_sq(r2_sub(x, m)) = r2_norm_sq(r2_sub(y, m))
// }
