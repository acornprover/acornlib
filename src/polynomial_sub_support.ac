from nat import Nat
from algebra.ring.ring import Ring
from semiring import Semiring
from polynomial import Polynomial, coeff_zero_at, coeff_zero_from, coeff_zero_from_at,
    polynomial_support_bounded_by, polynomial_support_bounded_by_apply, polynomial_neg_coeff,
    polynomial_sub_coeff

numerals Nat

/// A coefficient function vanishing from `n` onward gives a support bound.
///
/// `polynomial_support_bounded_by_apply` reads a bound; nothing built one from the pointwise
/// condition, which is what a bound on a combination of polynomials has to do.
theorem polynomial_support_bounded_by_intro[R: Semiring](p: Polynomial[R], n: Nat) {
    (forall(k: Nat) { not k < n implies p.coeff(k) = R.0 })
        implies polynomial_support_bounded_by(p, n)
} by {
    if forall(k: Nat) { not k < n implies p.coeff(k) = R.0 } {
        forall(k: Nat) {
            if not k < n {
                p.coeff(k) = R.0
                (coeff_zero_at(p.coeff, k) = (p.coeff(k) = R.0))
                coeff_zero_at(p.coeff, k)
            }
            (coeff_zero_from_at(p.coeff, n, k)
                = (not k < n implies coeff_zero_at(p.coeff, k)))
            coeff_zero_from_at(p.coeff, n, k)
        }
        (coeff_zero_from(p.coeff, n) = forall(j: Nat) {
            coeff_zero_from_at(p.coeff, n, j)
        })
        coeff_zero_from(p.coeff, n)
        (polynomial_support_bounded_by(p, n) = coeff_zero_from(p.coeff, n))
        polynomial_support_bounded_by(p, n)
    }
}

/// Negation preserves a support bound.
theorem polynomial_neg_support_bounded_by[R: Ring](p: Polynomial[R], n: Nat) {
    polynomial_support_bounded_by(p, n)
        implies polynomial_support_bounded_by(p.neg, n)
} by {
    if polynomial_support_bounded_by(p, n) {
        forall(k: Nat) {
            if not k < n {
                polynomial_support_bounded_by_apply(p, n, k)
                p.coeff(k) = R.0
                polynomial_neg_coeff(p, k)
                p.neg.coeff(k) = -p.coeff(k)
                p.neg.coeff(k) = -R.0
                -R.0 = R.0
                p.neg.coeff(k) = R.0
            }
            (not k < n implies p.neg.coeff(k) = R.0)
        }
        polynomial_support_bounded_by_intro(p.neg, n)
        polynomial_support_bounded_by(p.neg, n)
    }
}

/// A difference of polynomials is supported below a common bound.
///
/// The counterpart of `polynomial_add_support_bounded_by`, whose absence forced statements about
/// `p - c` to carry their hypotheses on the difference rather than on `p` and `c` separately.
theorem polynomial_sub_support_bounded_by[R: Ring](
    p: Polynomial[R], q: Polynomial[R], n: Nat
) {
    polynomial_support_bounded_by(p, n) and polynomial_support_bounded_by(q, n)
        implies polynomial_support_bounded_by(p.sub(q), n)
} by {
    if polynomial_support_bounded_by(p, n) and polynomial_support_bounded_by(q, n) {
        forall(k: Nat) {
            if not k < n {
                polynomial_support_bounded_by_apply(p, n, k)
                p.coeff(k) = R.0
                polynomial_support_bounded_by_apply(q, n, k)
                q.coeff(k) = R.0
                polynomial_sub_coeff(p, q, k)
                p.sub(q).coeff(k) = p.coeff(k) + -q.coeff(k)
                p.sub(q).coeff(k) = R.0 + -R.0
                -R.0 = R.0
                R.0 + R.0 = R.0
                p.sub(q).coeff(k) = R.0
            }
            (not k < n implies p.sub(q).coeff(k) = R.0)
        }
        polynomial_support_bounded_by_intro(p.sub(q), n)
        polynomial_support_bounded_by(p.sub(q), n)
    }
}
