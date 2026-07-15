/// Finite meet and join structures for ordered types.

from order import PartialOrder, lte_refl, lte_trans, lte_antisymm

/// A type with a binary meet operation.
typeclass A: Meet {
    /// The binary greatest lower bound operation.
    meet: (A, A) -> A
}

/// A type with a binary join operation.
typeclass A: Join {
    /// The binary least upper bound operation.
    join: (A, A) -> A
}

/// A partially ordered type with binary greatest lower bounds.
typeclass S: MeetSemilattice extends PartialOrder, Meet {
    /// A meet is below its left argument.
    meet_lte_left(a: S, b: S) {
        a.meet(b) <= a
    }

    /// A meet is below its right argument.
    meet_lte_right(a: S, b: S) {
        a.meet(b) <= b
    }

    /// Any common lower bound is below the meet.
    lte_meet_of_bounds(c: S, a: S, b: S) {
        c <= a and c <= b implies c <= a.meet(b)
    }
}

/// A partially ordered type with binary least upper bounds.
typeclass S: JoinSemilattice extends PartialOrder, Join {
    /// A join is above its left argument.
    lte_join_left(a: S, b: S) {
        a <= a.join(b)
    }

    /// A join is above its right argument.
    lte_join_right(a: S, b: S) {
        b <= a.join(b)
    }

    /// A join is below any common upper bound.
    join_lte_of_bounds(a: S, b: S, c: S) {
        a <= c and b <= c implies a.join(b) <= c
    }
}

/// A partially ordered type with binary meets and joins.
typeclass L: Lattice extends MeetSemilattice, JoinSemilattice {
}

/// A lattice where meets distribute over joins and joins distribute over meets.
typeclass L: DistribLattice extends Lattice {
    /// Meet distributes over join on the left.
    meet_join_distrib_left(a: L, b: L, c: L) {
        a.meet(b.join(c)) = a.meet(b).join(a.meet(c))
    }

    /// Join distributes over meet on the left.
    join_meet_distrib_left(a: L, b: L, c: L) {
        a.join(b.meet(c)) = a.join(b).meet(a.join(c))
    }
}

/// A meet is below its left argument.
theorem meet_lte_left[S: MeetSemilattice](a: S, b: S) {
    a.meet(b) <= a
}

/// A meet is below its left argument.
theorem meet_le_left[S: MeetSemilattice](a: S, b: S) {
    a.meet(b) <= a
} by {
    meet_lte_left(a, b)
}

/// A meet is below its right argument.
theorem meet_lte_right[S: MeetSemilattice](a: S, b: S) {
    a.meet(b) <= b
}

/// A meet is below its right argument.
theorem meet_le_right[S: MeetSemilattice](a: S, b: S) {
    a.meet(b) <= b
} by {
    meet_lte_right(a, b)
}

/// Any common lower bound is below the meet.
theorem lte_meet_of_bounds[S: MeetSemilattice](c: S, a: S, b: S) {
    c <= a and c <= b implies c <= a.meet(b)
}

/// Any common lower bound is below the meet.
theorem le_meet_of_le_left_of_le_right[S: MeetSemilattice](c: S, a: S, b: S) {
    c <= a and c <= b implies c <= a.meet(b)
} by {
    if c <= a and c <= b {
        lte_meet_of_bounds(c, a, b)
        c <= a.meet(b)
    }
}

/// The meet is the greatest lower bound of two elements.
theorem lte_meet_iff[S: MeetSemilattice](c: S, a: S, b: S) {
    c <= a.meet(b) = (c <= a and c <= b)
} by {
    if c <= a.meet(b) {
        meet_lte_left(a, b)
        lte_trans(c, a.meet(b), a)
        meet_lte_right(a, b)
        lte_trans(c, a.meet(b), b)
        c <= b
        c <= a and c <= b
    }
    if c <= a and c <= b {
        lte_meet_of_bounds(c, a, b)
        c <= a.meet(b)
    }
}

/// The meet is the greatest lower bound of two elements.
theorem le_meet_iff[S: MeetSemilattice](c: S, a: S, b: S) {
    c <= a.meet(b) = (c <= a and c <= b)
} by {
    lte_meet_iff(c, a, b)
}

/// A join is above its left argument.
theorem lte_join_left[S: JoinSemilattice](a: S, b: S) {
    a <= a.join(b)
}

/// A join is above its left argument.
theorem le_join_left[S: JoinSemilattice](a: S, b: S) {
    a <= a.join(b)
} by {
    lte_join_left(a, b)
}

/// A join is above its right argument.
theorem lte_join_right[S: JoinSemilattice](a: S, b: S) {
    b <= a.join(b)
}

/// A join is above its right argument.
theorem le_join_right[S: JoinSemilattice](a: S, b: S) {
    b <= a.join(b)
} by {
    lte_join_right(a, b)
}

/// A join is below any common upper bound.
theorem join_lte_of_bounds[S: JoinSemilattice](a: S, b: S, c: S) {
    a <= c and b <= c implies a.join(b) <= c
}

/// A join is below any common upper bound.
theorem join_le_of_le_left_of_le_right[S: JoinSemilattice](a: S, b: S, c: S) {
    a <= c and b <= c implies a.join(b) <= c
} by {
    if a <= c and b <= c {
        join_lte_of_bounds(a, b, c)
        a.join(b) <= c
    }
}

/// The join is the least upper bound of two elements.
theorem join_lte_iff[S: JoinSemilattice](a: S, b: S, c: S) {
    a.join(b) <= c = (a <= c and b <= c)
} by {
    if a.join(b) <= c {
        lte_join_left(a, b)
        lte_trans(a, a.join(b), c)
        lte_join_right(a, b)
        lte_trans(b, a.join(b), c)
        b <= c
        a <= c and b <= c
    }
    if a <= c and b <= c {
        join_lte_of_bounds(a, b, c)
        a.join(b) <= c
    }
}

/// The join is the least upper bound of two elements.
theorem join_le_iff[S: JoinSemilattice](a: S, b: S, c: S) {
    a.join(b) <= c = (a <= c and b <= c)
} by {
    join_lte_iff(a, b, c)
}

/// Meet distributes over join on the left.
theorem meet_join_distrib_left[S: DistribLattice](a: S, b: S, c: S) {
    a.meet(b.join(c)) = a.meet(b).join(a.meet(c))
}

/// Join distributes over meet on the left.
theorem join_meet_distrib_left[S: DistribLattice](a: S, b: S, c: S) {
    a.join(b.meet(c)) = a.join(b).meet(a.join(c))
}

/// Meet is commutative.
theorem meet_comm[S: MeetSemilattice](a: S, b: S) {
    a.meet(b) = b.meet(a)
} by {
    meet_lte_right(a, b)
    meet_lte_left(a, b)
    lte_meet_of_bounds(a.meet(b), b, a)
    meet_lte_right(b, a)
    meet_lte_left(b, a)
    lte_meet_of_bounds(b.meet(a), a, b)
    lte_antisymm(a.meet(b), b.meet(a))
}

/// Join is commutative.
theorem join_comm[S: JoinSemilattice](a: S, b: S) {
    a.join(b) = b.join(a)
} by {
    lte_join_right(a, b)
    lte_join_left(a, b)
    join_lte_of_bounds(a, b, b.join(a))
    lte_join_right(b, a)
    lte_join_left(b, a)
    join_lte_of_bounds(b, a, a.join(b))
    lte_antisymm(a.join(b), b.join(a))
}

/// Meet is idempotent.
theorem meet_idem[S: MeetSemilattice](a: S) {
    a.meet(a) = a
} by {
    meet_lte_left(a, a)
    lte_refl(a)
    lte_meet_of_bounds(a, a, a)
    lte_antisymm(a.meet(a), a)
}

/// Join is idempotent.
theorem join_idem[S: JoinSemilattice](a: S) {
    a.join(a) = a
} by {
    lte_join_left(a, a)
    lte_refl(a)
    join_lte_of_bounds(a, a, a)
    lte_antisymm(a.join(a), a)
}

/// Meet is associative.
theorem meet_assoc[S: MeetSemilattice](a: S, b: S, c: S) {
    a.meet(b).meet(c) = a.meet(b.meet(c))
} by {
    meet_lte_left(a.meet(b), c)
    meet_lte_left(a, b)
    lte_trans(a.meet(b).meet(c), a.meet(b), a)
    meet_lte_right(a, b)
    lte_trans(a.meet(b).meet(c), a.meet(b), b)
    meet_lte_right(a.meet(b), c)
    lte_meet_of_bounds(a.meet(b).meet(c), b, c)
    lte_meet_of_bounds(a.meet(b).meet(c), a, b.meet(c))
    meet_lte_left(a, b.meet(c))
    meet_lte_right(a, b.meet(c))
    meet_lte_left(b, c)
    lte_trans(a.meet(b.meet(c)), b.meet(c), b)
    meet_lte_right(b, c)
    lte_trans(a.meet(b.meet(c)), b.meet(c), c)
    lte_meet_of_bounds(a.meet(b.meet(c)), a, b)
    lte_meet_of_bounds(a.meet(b.meet(c)), a.meet(b), c)
    lte_antisymm(a.meet(b).meet(c), a.meet(b.meet(c)))
}

/// Join is associative.
theorem join_assoc[S: JoinSemilattice](a: S, b: S, c: S) {
    a.join(b).join(c) = a.join(b.join(c))
} by {
    lte_join_left(a, b)
    lte_join_left(a.join(b), c)
    lte_trans(a, a.join(b), a.join(b).join(c))
    lte_join_right(a, b)
    lte_join_left(a.join(b), c)
    lte_trans(b, a.join(b), a.join(b).join(c))
    lte_join_right(a.join(b), c)
    join_lte_of_bounds(b, c, a.join(b).join(c))
    join_lte_of_bounds(a, b.join(c), a.join(b).join(c))
    lte_join_left(a, b.join(c))
    lte_join_right(a, b.join(c))
    lte_join_left(b, c)
    lte_trans(b, b.join(c), a.join(b.join(c)))
    lte_join_right(b, c)
    lte_trans(c, b.join(c), a.join(b.join(c)))
    join_lte_of_bounds(a, b, a.join(b.join(c)))
    join_lte_of_bounds(a.join(b), c, a.join(b.join(c)))
    lte_antisymm(a.join(b).join(c), a.join(b.join(c)))
}

/// Meet is monotone in both arguments.
theorem meet_lte_meet[S: MeetSemilattice](a: S, b: S, c: S, d: S) {
    a <= b and c <= d implies a.meet(c) <= b.meet(d)
} by {
    if a <= b and c <= d {
        meet_lte_left(a, c)
        lte_trans(a.meet(c), a, b)
        meet_lte_right(a, c)
        lte_trans(a.meet(c), c, d)
        lte_meet_of_bounds(a.meet(c), b, d)
        a.meet(c) <= b.meet(d)
    }
}

/// Join is monotone in both arguments.
theorem join_lte_join[S: JoinSemilattice](a: S, b: S, c: S, d: S) {
    a <= b and c <= d implies a.join(c) <= b.join(d)
} by {
    if a <= b and c <= d {
        lte_join_left(b, d)
        lte_trans(a, b, b.join(d))
        lte_join_right(b, d)
        lte_trans(c, d, b.join(d))
        join_lte_of_bounds(a, c, b.join(d))
        a.join(c) <= b.join(d)
    }
}

/// Meet is monotone in the left argument.
theorem meet_lte_meet_left[S: MeetSemilattice](a: S, b: S, c: S) {
    a <= b implies a.meet(c) <= b.meet(c)
} by {
    if a <= b {
        lte_refl(c)
        meet_lte_meet(a, b, c, c)
        a.meet(c) <= b.meet(c)
    }
}

/// Meet is monotone in the left argument.
theorem meet_le_meet_left[S: MeetSemilattice](a: S, b: S, c: S) {
    a <= b implies a.meet(c) <= b.meet(c)
} by {
    if a <= b {
        meet_lte_meet_left(a, b, c)
        a.meet(c) <= b.meet(c)
    }
}

/// Meet is monotone in the right argument.
theorem meet_lte_meet_right[S: MeetSemilattice](a: S, b: S, c: S) {
    b <= c implies a.meet(b) <= a.meet(c)
} by {
    if b <= c {
        lte_refl(a)
        meet_lte_meet(a, a, b, c)
        a.meet(b) <= a.meet(c)
    }
}

/// Meet is monotone in the right argument.
theorem meet_le_meet_right[S: MeetSemilattice](a: S, b: S, c: S) {
    b <= c implies a.meet(b) <= a.meet(c)
} by {
    if b <= c {
        meet_lte_meet_right(a, b, c)
        a.meet(b) <= a.meet(c)
    }
}

/// Join is monotone in the left argument.
theorem join_lte_join_left[S: JoinSemilattice](a: S, b: S, c: S) {
    a <= b implies a.join(c) <= b.join(c)
} by {
    if a <= b {
        lte_refl(c)
        join_lte_join(a, b, c, c)
        a.join(c) <= b.join(c)
    }
}

/// Join is monotone in the left argument.
theorem join_le_join_left[S: JoinSemilattice](a: S, b: S, c: S) {
    a <= b implies a.join(c) <= b.join(c)
} by {
    if a <= b {
        join_lte_join_left(a, b, c)
        a.join(c) <= b.join(c)
    }
}

/// Join is monotone in the right argument.
theorem join_lte_join_right[S: JoinSemilattice](a: S, b: S, c: S) {
    b <= c implies a.join(b) <= a.join(c)
} by {
    if b <= c {
        lte_refl(a)
        join_lte_join(a, a, b, c)
        a.join(b) <= a.join(c)
    }
}

/// Join is monotone in the right argument.
theorem join_le_join_right[S: JoinSemilattice](a: S, b: S, c: S) {
    b <= c implies a.join(b) <= a.join(c)
} by {
    if b <= c {
        join_lte_join_right(a, b, c)
        a.join(b) <= a.join(c)
    }
}

/// A meet equals its left argument exactly when the left argument is below the right.
theorem meet_eq_left_iff_lte[S: MeetSemilattice](a: S, b: S) {
    a.meet(b) = a = (a <= b)
} by {
    if a.meet(b) = a {
        meet_lte_right(a, b)
        a <= b
    }
    if a <= b {
        meet_lte_left(a, b)
        lte_refl(a)
        lte_meet_of_bounds(a, a, b)
        lte_antisymm(a.meet(b), a)
        a.meet(b) = a
    }
}

/// A meet equals its right argument exactly when the right argument is below the left.
theorem meet_eq_right_iff_lte[S: MeetSemilattice](a: S, b: S) {
    a.meet(b) = b = (b <= a)
} by {
    meet_comm(a, b)
    meet_eq_left_iff_lte(b, a)
}

/// A join equals its left argument exactly when the right argument is below the left.
theorem join_eq_left_iff_lte[S: JoinSemilattice](a: S, b: S) {
    a.join(b) = a = (b <= a)
} by {
    if a.join(b) = a {
        lte_join_right(a, b)
        b <= a
    }
    if b <= a {
        lte_join_left(a, b)
        lte_refl(a)
        join_lte_of_bounds(a, b, a)
        lte_antisymm(a.join(b), a)
        a.join(b) = a
    }
}

/// A join equals its right argument exactly when the left argument is below the right.
theorem join_eq_right_iff_lte[S: JoinSemilattice](a: S, b: S) {
    a.join(b) = b = (a <= b)
} by {
    join_comm(a, b)
    join_eq_left_iff_lte(b, a)
}

/// A meet equals its left argument when the left argument is below the right.
theorem meet_eq_left_of_lte[S: MeetSemilattice](a: S, b: S) {
    a <= b implies a.meet(b) = a
} by {
    if a <= b {
        meet_eq_left_iff_lte(a, b)
        a.meet(b) = a
    }
}

/// A meet equals its left argument when the left argument is below the right.
theorem meet_eq_left_of_le[S: MeetSemilattice](a: S, b: S) {
    a <= b implies a.meet(b) = a
} by {
    if a <= b {
        meet_eq_left_of_lte(a, b)
        a.meet(b) = a
    }
}

/// A meet equals its right argument when the right argument is below the left.
theorem meet_eq_right_of_lte[S: MeetSemilattice](a: S, b: S) {
    b <= a implies a.meet(b) = b
} by {
    if b <= a {
        meet_eq_right_iff_lte(a, b)
        a.meet(b) = b
    }
}

/// A meet equals its right argument when the right argument is below the left.
theorem meet_eq_right_of_le[S: MeetSemilattice](a: S, b: S) {
    b <= a implies a.meet(b) = b
} by {
    if b <= a {
        meet_eq_right_of_lte(a, b)
        a.meet(b) = b
    }
}

/// A join equals its left argument when the right argument is below the left.
theorem join_eq_left_of_lte[S: JoinSemilattice](a: S, b: S) {
    b <= a implies a.join(b) = a
} by {
    if b <= a {
        join_eq_left_iff_lte(a, b)
        a.join(b) = a
    }
}

/// A join equals its left argument when the right argument is below the left.
theorem join_eq_left_of_le[S: JoinSemilattice](a: S, b: S) {
    b <= a implies a.join(b) = a
} by {
    if b <= a {
        join_eq_left_of_lte(a, b)
        a.join(b) = a
    }
}

/// A join equals its right argument when the left argument is below the right.
theorem join_eq_right_of_lte[S: JoinSemilattice](a: S, b: S) {
    a <= b implies a.join(b) = b
} by {
    if a <= b {
        join_eq_right_iff_lte(a, b)
        a.join(b) = b
    }
}

/// A join equals its right argument when the left argument is below the right.
theorem join_eq_right_of_le[S: JoinSemilattice](a: S, b: S) {
    a <= b implies a.join(b) = b
} by {
    if a <= b {
        join_eq_right_of_lte(a, b)
        a.join(b) = b
    }
}

/// An infimum equals its left argument when the left argument is below the right.
theorem inf_eq_left_of_le[S: MeetSemilattice](a: S, b: S) {
    a <= b implies a.meet(b) = a
} by {
    if a <= b {
        meet_eq_left_of_lte(a, b)
        a.meet(b) = a
    }
}

/// An infimum equals its right argument when the right argument is below the left.
theorem inf_eq_right_of_le[S: MeetSemilattice](a: S, b: S) {
    b <= a implies a.meet(b) = b
} by {
    if b <= a {
        meet_eq_right_of_lte(a, b)
        a.meet(b) = b
    }
}

/// A supremum equals its left argument when the right argument is below the left.
theorem sup_eq_left_of_le[S: JoinSemilattice](a: S, b: S) {
    b <= a implies a.join(b) = a
} by {
    if b <= a {
        join_eq_left_of_lte(a, b)
        a.join(b) = a
    }
}

/// A supremum equals its right argument when the left argument is below the right.
theorem sup_eq_right_of_le[S: JoinSemilattice](a: S, b: S) {
    a <= b implies a.join(b) = b
} by {
    if a <= b {
        join_eq_right_of_lte(a, b)
        a.join(b) = b
    }
}

/// Meet absorbs join on the right.
theorem meet_absorb_join[S: Lattice](a: S, b: S) {
    a.meet(a.join(b)) = a
} by {
    meet_lte_left(a, a.join(b))
    lte_refl(a)
    lte_join_left(a, b)
    lte_meet_of_bounds(a, a, a.join(b))
    lte_antisymm(a.meet(a.join(b)), a)
}

/// Join absorbs meet on the right.
theorem join_absorb_meet[S: Lattice](a: S, b: S) {
    a.join(a.meet(b)) = a
} by {
    lte_join_left(a, a.meet(b))
    lte_refl(a)
    meet_lte_left(a, b)
    join_lte_of_bounds(a, a.meet(b), a)
    lte_antisymm(a.join(a.meet(b)), a)
}

/// Meet absorbs join on the left.
theorem meet_absorb_join_left[S: Lattice](a: S, b: S) {
    a.join(b).meet(a) = a
} by {
    meet_comm(a.join(b), a)
    meet_absorb_join(a, b)
}

/// Join absorbs meet on the left.
theorem join_absorb_meet_left[S: Lattice](a: S, b: S) {
    a.meet(b).join(a) = a
} by {
    join_comm(a.meet(b), a)
    join_absorb_meet(a, b)
}

/// Meet distributes over join on the right.
theorem meet_join_distrib_right[S: DistribLattice](a: S, b: S, c: S) {
    a.join(b).meet(c) = a.meet(c).join(b.meet(c))
} by {
    meet_comm(a.join(b), c)
    meet_join_distrib_left(c, a, b)
    meet_comm(c, a)
    meet_comm(c, b)
}

/// Join distributes over meet on the right.
theorem join_meet_distrib_right[S: DistribLattice](a: S, b: S, c: S) {
    a.meet(b).join(c) = a.join(c).meet(b.join(c))
} by {
    join_comm(a.meet(b), c)
    join_meet_distrib_left(c, a, b)
    join_comm(c, a)
    join_comm(c, b)
}

/// An infimum is below its left argument.
theorem inf_le_left[S: MeetSemilattice](a: S, b: S) {
    a.meet(b) <= a
} by {
    meet_lte_left(a, b)
}

/// An infimum is below its right argument.
theorem inf_le_right[S: MeetSemilattice](a: S, b: S) {
    a.meet(b) <= b
} by {
    meet_lte_right(a, b)
}

/// A common lower bound is below the infimum.
theorem le_inf[S: MeetSemilattice](c: S, a: S, b: S) {
    c <= a and c <= b implies c <= a.meet(b)
} by {
    if c <= a and c <= b {
        lte_meet_of_bounds(c, a, b)
        c <= a.meet(b)
    }
}

/// Being below an infimum is equivalent to being below each argument.
theorem le_inf_iff[S: MeetSemilattice](c: S, a: S, b: S) {
    c <= a.meet(b) = (c <= a and c <= b)
} by {
    lte_meet_iff(c, a, b)
}

/// The left argument is below the supremum.
theorem le_sup_left[S: JoinSemilattice](a: S, b: S) {
    a <= a.join(b)
} by {
    lte_join_left(a, b)
}

/// The right argument is below the supremum.
theorem le_sup_right[S: JoinSemilattice](a: S, b: S) {
    b <= a.join(b)
} by {
    lte_join_right(a, b)
}

/// A supremum is below every common upper bound.
theorem sup_le[S: JoinSemilattice](a: S, b: S, c: S) {
    a <= c and b <= c implies a.join(b) <= c
} by {
    if a <= c and b <= c {
        join_lte_of_bounds(a, b, c)
        a.join(b) <= c
    }
}

/// A supremum is below a point exactly when each argument is below that point.
theorem sup_le_iff[S: JoinSemilattice](a: S, b: S, c: S) {
    a.join(b) <= c = (a <= c and b <= c)
} by {
    join_lte_iff(a, b, c)
}

/// Infimum is commutative.
theorem inf_comm[S: MeetSemilattice](a: S, b: S) {
    a.meet(b) = b.meet(a)
} by {
    meet_comm(a, b)
}

/// Supremum is commutative.
theorem sup_comm[S: JoinSemilattice](a: S, b: S) {
    a.join(b) = b.join(a)
} by {
    join_comm(a, b)
}

/// Infimum is associative.
theorem inf_assoc[S: MeetSemilattice](a: S, b: S, c: S) {
    a.meet(b).meet(c) = a.meet(b.meet(c))
} by {
    meet_assoc(a, b, c)
}

/// Supremum is associative.
theorem sup_assoc[S: JoinSemilattice](a: S, b: S, c: S) {
    a.join(b).join(c) = a.join(b.join(c))
} by {
    join_assoc(a, b, c)
}

/// Infimum is idempotent.
theorem inf_idem[S: MeetSemilattice](a: S) {
    a.meet(a) = a
} by {
    meet_idem(a)
}

/// Supremum is idempotent.
theorem sup_idem[S: JoinSemilattice](a: S) {
    a.join(a) = a
} by {
    join_idem(a)
}

/// Infimum is monotone in both arguments.
theorem inf_le_inf[S: MeetSemilattice](a: S, b: S, c: S, d: S) {
    a <= b and c <= d implies a.meet(c) <= b.meet(d)
} by {
    if a <= b and c <= d {
        meet_lte_meet(a, b, c, d)
        a.meet(c) <= b.meet(d)
    }
}

/// Supremum is monotone in both arguments.
theorem sup_le_sup[S: JoinSemilattice](a: S, b: S, c: S, d: S) {
    a <= b and c <= d implies a.join(c) <= b.join(d)
} by {
    if a <= b and c <= d {
        join_lte_join(a, b, c, d)
        a.join(c) <= b.join(d)
    }
}

/// Infimum is monotone in the left argument.
theorem inf_le_inf_left[S: MeetSemilattice](a: S, b: S, c: S) {
    a <= b implies a.meet(c) <= b.meet(c)
} by {
    if a <= b {
        meet_lte_meet_left(a, b, c)
        a.meet(c) <= b.meet(c)
    }
}

/// Infimum is monotone in the right argument.
theorem inf_le_inf_right[S: MeetSemilattice](a: S, b: S, c: S) {
    b <= c implies a.meet(b) <= a.meet(c)
} by {
    if b <= c {
        meet_lte_meet_right(a, b, c)
        a.meet(b) <= a.meet(c)
    }
}

/// Supremum is monotone in the left argument.
theorem sup_le_sup_left[S: JoinSemilattice](a: S, b: S, c: S) {
    a <= b implies a.join(c) <= b.join(c)
} by {
    if a <= b {
        join_lte_join_left(a, b, c)
        a.join(c) <= b.join(c)
    }
}

/// Supremum is monotone in the right argument.
theorem sup_le_sup_right[S: JoinSemilattice](a: S, b: S, c: S) {
    b <= c implies a.join(b) <= a.join(c)
} by {
    if b <= c {
        join_lte_join_right(a, b, c)
        a.join(b) <= a.join(c)
    }
}

/// An infimum equals its left argument exactly when the left argument is below the right.
theorem inf_eq_left_iff[S: MeetSemilattice](a: S, b: S) {
    a.meet(b) = a = (a <= b)
} by {
    meet_eq_left_iff_lte(a, b)
}

/// An infimum equals its right argument exactly when the right argument is below the left.
theorem inf_eq_right_iff[S: MeetSemilattice](a: S, b: S) {
    a.meet(b) = b = (b <= a)
} by {
    meet_eq_right_iff_lte(a, b)
}

/// A supremum equals its left argument exactly when the right argument is below the left.
theorem sup_eq_left_iff[S: JoinSemilattice](a: S, b: S) {
    a.join(b) = a = (b <= a)
} by {
    join_eq_left_iff_lte(a, b)
}

/// A supremum equals its right argument exactly when the left argument is below the right.
theorem sup_eq_right_iff[S: JoinSemilattice](a: S, b: S) {
    a.join(b) = b = (a <= b)
} by {
    join_eq_right_iff_lte(a, b)
}

/// Infimum absorbs a supremum on the right.
theorem inf_sup_self[S: Lattice](a: S, b: S) {
    a.meet(a.join(b)) = a
} by {
    meet_absorb_join(a, b)
}

/// Supremum absorbs an infimum on the right.
theorem sup_inf_self[S: Lattice](a: S, b: S) {
    a.join(a.meet(b)) = a
} by {
    join_absorb_meet(a, b)
}

/// Infimum distributes over supremum on the left.
theorem inf_sup_left[S: DistribLattice](a: S, b: S, c: S) {
    a.meet(b.join(c)) = a.meet(b).join(a.meet(c))
} by {
    meet_join_distrib_left(a, b, c)
}

/// Supremum distributes over infimum on the left.
theorem sup_inf_left[S: DistribLattice](a: S, b: S, c: S) {
    a.join(b.meet(c)) = a.join(b).meet(a.join(c))
} by {
    join_meet_distrib_left(a, b, c)
}

/// Infimum distributes over supremum on the right.
theorem inf_sup_right[S: DistribLattice](a: S, b: S, c: S) {
    a.join(b).meet(c) = a.meet(c).join(b.meet(c))
} by {
    meet_join_distrib_right(a, b, c)
}

/// Supremum distributes over infimum on the right.
theorem sup_inf_right[S: DistribLattice](a: S, b: S, c: S) {
    a.meet(b).join(c) = a.join(c).meet(b.join(c))
} by {
    join_meet_distrib_right(a, b, c)
}

/// Meet is associative in the reversed nesting.
theorem meet_assoc_rev[S: MeetSemilattice](a: S, b: S, c: S) {
    a.meet(b.meet(c)) = a.meet(b).meet(c)
} by {
    meet_assoc(a, b, c)
}

/// Join is associative in the reversed nesting.
theorem join_assoc_rev[S: JoinSemilattice](a: S, b: S, c: S) {
    a.join(b.join(c)) = a.join(b).join(c)
} by {
    join_assoc(a, b, c)
}

/// The outer left argument of a nested meet may be exchanged.
theorem meet_left_comm[S: MeetSemilattice](a: S, b: S, c: S) {
    a.meet(b.meet(c)) = b.meet(a.meet(c))
} by {
    meet_assoc_rev(a, b, c)
    meet_comm(a, b)
    meet_assoc(b, a, c)
}

/// The outer left argument of a nested join may be exchanged.
theorem join_left_comm[S: JoinSemilattice](a: S, b: S, c: S) {
    a.join(b.join(c)) = b.join(a.join(c))
} by {
    join_assoc_rev(a, b, c)
    join_comm(a, b)
    join_assoc(b, a, c)
}

/// The two right arguments of an iterated meet may be exchanged.
theorem meet_right_comm[S: MeetSemilattice](a: S, b: S, c: S) {
    a.meet(b).meet(c) = a.meet(c).meet(b)
} by {
    meet_assoc(a, b, c)
    meet_comm(b, c)
    meet_assoc_rev(a, c, b)
}

/// The two right arguments of an iterated join may be exchanged.
theorem join_right_comm[S: JoinSemilattice](a: S, b: S, c: S) {
    a.join(b).join(c) = a.join(c).join(b)
} by {
    join_assoc(a, b, c)
    join_comm(b, c)
    join_assoc_rev(a, c, b)
}

/// A meet absorbs a join with the same right argument.
theorem meet_absorb_join_right[S: Lattice](a: S, b: S) {
    a.meet(b.join(a)) = a
} by {
    join_comm(b, a)
    meet_absorb_join(a, b)
}

/// A join absorbs a meet with the same right argument.
theorem join_absorb_meet_right[S: Lattice](a: S, b: S) {
    a.join(b.meet(a)) = a
} by {
    meet_comm(b, a)
    join_absorb_meet(a, b)
}

/// Infimum is associative in the reversed nesting.
theorem inf_assoc_rev[S: MeetSemilattice](a: S, b: S, c: S) {
    a.meet(b.meet(c)) = a.meet(b).meet(c)
} by {
    meet_assoc_rev(a, b, c)
}

/// Supremum is associative in the reversed nesting.
theorem sup_assoc_rev[S: JoinSemilattice](a: S, b: S, c: S) {
    a.join(b.join(c)) = a.join(b).join(c)
} by {
    join_assoc_rev(a, b, c)
}

/// The outer left argument of a nested infimum may be exchanged.
theorem inf_left_comm[S: MeetSemilattice](a: S, b: S, c: S) {
    a.meet(b.meet(c)) = b.meet(a.meet(c))
} by {
    meet_left_comm(a, b, c)
}

/// The outer left argument of a nested supremum may be exchanged.
theorem sup_left_comm[S: JoinSemilattice](a: S, b: S, c: S) {
    a.join(b.join(c)) = b.join(a.join(c))
} by {
    join_left_comm(a, b, c)
}

/// The two right arguments of an iterated infimum may be exchanged.
theorem inf_right_comm[S: MeetSemilattice](a: S, b: S, c: S) {
    a.meet(b).meet(c) = a.meet(c).meet(b)
} by {
    meet_right_comm(a, b, c)
}

/// The two right arguments of an iterated supremum may be exchanged.
theorem sup_right_comm[S: JoinSemilattice](a: S, b: S, c: S) {
    a.join(b).join(c) = a.join(c).join(b)
} by {
    join_right_comm(a, b, c)
}

/// Infimum absorbs a supremum with the same right argument.
theorem inf_sup_self_right[S: Lattice](a: S, b: S) {
    a.meet(b.join(a)) = a
} by {
    meet_absorb_join_right(a, b)
}

/// Supremum absorbs an infimum with the same right argument.
theorem sup_inf_self_right[S: Lattice](a: S, b: S) {
    a.join(b.meet(a)) = a
} by {
    join_absorb_meet_right(a, b)
}
