from nat import Nat
from int import Int
from rat import Rat
from data.basic.functions import is_injective_fn
from list import List, map, map_contains, map_length, injective_map_is_unique
from polynomial import Polynomial, polynomial_eval, polynomial_map,
    polynomial_map_coeff_apply, polynomial_support_bounded_by,
    polynomial_map_support_bounded_by, polynomial_ext_pointwise, polynomial_zero_coeff,
    polynomial_roots_on_list, polynomial_root_list_length_lt_support_bound,
    polynomial_eval_map, polynomial_eval_sub, polynomial_eval_constant
from data.rat.rat_int_hom import int_to_rat_hom, int_to_rat_hom_apply, int_to_rat_is_injective,
    int_zero_iff_rat_zero

numerals Nat

/// The rational polynomial obtained from an integer one.
define rationalise(p: Polynomial[Int]) -> Polynomial[Rat] {
    polynomial_map(int_to_rat_hom, p)
}

/// The coefficients of the rationalised polynomial are the images of the originals.
theorem rationalise_coeff(p: Polynomial[Int], n: Nat) {
    rationalise(p).coeff(n) = Rat.from_int(p.coeff(n))
} by {
    polynomial_map_coeff_apply(int_to_rat_hom, p, n)
    rationalise(p).coeff(n) = int_to_rat_hom.hom(p.coeff(n))
    int_to_rat_hom_apply(p.coeff(n))
    int_to_rat_hom.hom(p.coeff(n)) = Rat.from_int(p.coeff(n))
}

/// Rationalising a nonzero integer polynomial gives a nonzero rational one.
///
/// Some coefficient of the original is nonzero, and the embedding is injective, so the
/// corresponding coefficient of the image is nonzero too.
theorem rationalise_ne_zero(p: Polynomial[Int]) {
    p != Polynomial[Int].zero implies rationalise(p) != Polynomial[Rat].zero
} by {
    if p != Polynomial[Int].zero {
        if rationalise(p) = Polynomial[Rat].zero {
            forall(n: Nat) {
                rationalise_coeff(p, n)
                rationalise(p).coeff(n) = Rat.from_int(p.coeff(n))
                polynomial_zero_coeff[Rat](n)
                Polynomial[Rat].zero.coeff(n) = Rat.0
                Rat.from_int(p.coeff(n)) = Rat.0
                int_zero_iff_rat_zero(p.coeff(n))
                p.coeff(n) = Int.0
                polynomial_zero_coeff[Int](n)
                Polynomial[Int].zero.coeff(n) = Int.0
                p.coeff(n) = Polynomial[Int].zero.coeff(n)
            }
            polynomial_ext_pointwise(p, Polynomial[Int].zero)
            p = Polynomial[Int].zero
            false
        }
        rationalise(p) != Polynomial[Rat].zero
    }
}

/// A support bound survives rationalising.
theorem rationalise_support_bounded_by(p: Polynomial[Int], n: Nat) {
    polynomial_support_bounded_by(p, n)
        implies polynomial_support_bounded_by(rationalise(p), n)
} by {
    if polynomial_support_bounded_by(p, n) {
        polynomial_map_support_bounded_by(int_to_rat_hom, p, n)
        polynomial_support_bounded_by(polynomial_map(int_to_rat_hom, p), n)
        polynomial_support_bounded_by(rationalise(p), n)
    }
}

/// True if every listed integer is a root of the integer polynomial.
define int_roots_on_list(p: Polynomial[Int], roots: List[Int]) -> Bool {
    forall(x: Int) {
        roots.contains(x) implies polynomial_eval(p, x) = Int.0
    }
}

/// Integer roots become rational roots of the rationalised polynomial.
///
/// Evaluation commutes with the homomorphism, and an integer vanishes exactly when its image
/// does, so the two root conditions match term by term.
theorem int_roots_rationalise(p: Polynomial[Int], roots: List[Int]) {
    int_roots_on_list(p, roots)
        implies polynomial_roots_on_list(rationalise(p), map(roots, Rat.from_int))
} by {
    if int_roots_on_list(p, roots) {
        forall(y: Rat) {
            if map(roots, Rat.from_int).contains(y) {
                map_contains(roots, Rat.from_int, y)
                exists(x: Int) {
                    roots.contains(x) and Rat.from_int(x) = y
                }
                let (x: Int) satisfy {
                    roots.contains(x) and Rat.from_int(x) = y
                }
                int_roots_on_list(p, roots) = forall(z: Int) {
                    roots.contains(z) implies polynomial_eval(p, z) = Int.0
                }
                forall(z: Int) {
                    roots.contains(z) implies polynomial_eval(p, z) = Int.0
                }
                (roots.contains(x) implies polynomial_eval(p, x) = Int.0)
                polynomial_eval(p, x) = Int.0
                polynomial_eval_map(int_to_rat_hom, p, x)
                (polynomial_eval(polynomial_map(int_to_rat_hom, p), int_to_rat_hom.hom(x))
                    = int_to_rat_hom.hom(polynomial_eval(p, x)))
                int_to_rat_hom_apply(x)
                int_to_rat_hom.hom(x) = Rat.from_int(x)
                int_to_rat_hom_apply(polynomial_eval(p, x))
                int_to_rat_hom.hom(polynomial_eval(p, x)) = Rat.from_int(polynomial_eval(p, x))
                Rat.from_int(Int.0) = Rat.0
                polynomial_eval(rationalise(p), Rat.from_int(x)) = Rat.0
                polynomial_eval(rationalise(p), y) = Rat.0
            }
            (map(roots, Rat.from_int).contains(y)
                implies polynomial_eval(rationalise(p), y) = Rat.0)
        }
        polynomial_roots_on_list(rationalise(p), map(roots, Rat.from_int)) = forall(y: Rat) {
            map(roots, Rat.from_int).contains(y) implies polynomial_eval(rationalise(p), y) = Rat.0
        }
        polynomial_roots_on_list(rationalise(p), map(roots, Rat.from_int))
    }
}

/// A nonzero integer polynomial supported below `n` has fewer than `n` distinct integer roots.
///
/// The bound in `src/polynomial/root_bound.ac` is stated over a `Field`, and the integers are
/// not one. Transporting along the embedding in the rationals recovers it: the rationalised
/// polynomial is nonzero with the same support bound, and the embedded root list is still
/// distinct because the embedding is injective.
theorem int_polynomial_root_list_length_lt_support_bound(
    p: Polynomial[Int], roots: List[Int], n: Nat
) {
    polynomial_support_bounded_by(p, n) and p != Polynomial[Int].zero
        and roots.is_unique and int_roots_on_list(p, roots)
        implies roots.length < n
} by {
    if polynomial_support_bounded_by(p, n) and p != Polynomial[Int].zero
        and roots.is_unique and int_roots_on_list(p, roots) {
        rationalise_support_bounded_by(p, n)
        polynomial_support_bounded_by(rationalise(p), n)
        rationalise_ne_zero(p)
        rationalise(p) != Polynomial[Rat].zero
        int_to_rat_is_injective
        is_injective_fn(Rat.from_int)
        injective_map_is_unique(roots, Rat.from_int)
        map(roots, Rat.from_int).is_unique
        int_roots_rationalise(p, roots)
        polynomial_roots_on_list(rationalise(p), map(roots, Rat.from_int))
        polynomial_root_list_length_lt_support_bound(rationalise(p),
            map(roots, Rat.from_int), n)
        map(roots, Rat.from_int).length < n
        map_length(roots, Rat.from_int)
        map(roots, Rat.from_int).length = roots.length
        roots.length < n
    }
}

/// True if every listed integer is a solution of `p(x) = c`.
define int_solutions_on_list(p: Polynomial[Int], c: Int, roots: List[Int]) -> Bool {
    forall(x: Int) {
        roots.contains(x) implies polynomial_eval(p, x) = c
    }
}

/// Solutions of `p(x) = c` are roots of `p - c`.
theorem int_solutions_are_roots(p: Polynomial[Int], c: Int, roots: List[Int]) {
    int_solutions_on_list(p, c, roots)
        implies int_roots_on_list(p.sub(Polynomial[Int].constant(c)), roots)
} by {
    if int_solutions_on_list(p, c, roots) {
        forall(x: Int) {
            if roots.contains(x) {
                int_solutions_on_list(p, c, roots) = forall(z: Int) {
                    roots.contains(z) implies polynomial_eval(p, z) = c
                }
                forall(z: Int) {
                    roots.contains(z) implies polynomial_eval(p, z) = c
                }
                (roots.contains(x) implies polynomial_eval(p, x) = c)
                polynomial_eval(p, x) = c
                polynomial_eval_sub(p, Polynomial[Int].constant(c), x)
                (polynomial_eval(p.sub(Polynomial[Int].constant(c)), x)
                    = polynomial_eval(p, x) - polynomial_eval(Polynomial[Int].constant(c), x))
                polynomial_eval_constant(c, x)
                polynomial_eval(Polynomial[Int].constant(c), x) = c
                c - c = Int.0
                polynomial_eval(p.sub(Polynomial[Int].constant(c)), x) = Int.0
            }
            (roots.contains(x)
                implies polynomial_eval(p.sub(Polynomial[Int].constant(c)), x) = Int.0)
        }
        int_roots_on_list(p.sub(Polynomial[Int].constant(c)), roots) = forall(x: Int) {
            roots.contains(x) implies polynomial_eval(p.sub(Polynomial[Int].constant(c)), x) = Int.0
        }
        int_roots_on_list(p.sub(Polynomial[Int].constant(c)), roots)
    }
}

/// An integer polynomial takes each value only finitely often.
///
/// The solutions of `p(x) = c` are the roots of `p - c`, so the root bound applies to them.
/// The hypotheses are on `p - c` rather than on `p`, because that is what the bound needs and
/// `src/polynomial/` has no support bound for a difference; supplying one for the difference is
/// no harder at a call site, where the degree is known.
theorem int_polynomial_value_list_length_lt_support_bound(
    p: Polynomial[Int], c: Int, roots: List[Int], n: Nat
) {
    polynomial_support_bounded_by(p.sub(Polynomial[Int].constant(c)), n)
        and p.sub(Polynomial[Int].constant(c)) != Polynomial[Int].zero
        and roots.is_unique and int_solutions_on_list(p, c, roots)
        implies roots.length < n
} by {
    if polynomial_support_bounded_by(p.sub(Polynomial[Int].constant(c)), n)
        and p.sub(Polynomial[Int].constant(c)) != Polynomial[Int].zero
        and roots.is_unique and int_solutions_on_list(p, c, roots) {
        int_solutions_are_roots(p, c, roots)
        int_roots_on_list(p.sub(Polynomial[Int].constant(c)), roots)
        int_polynomial_root_list_length_lt_support_bound(
            p.sub(Polynomial[Int].constant(c)), roots, n)
        roots.length < n
    }
}
