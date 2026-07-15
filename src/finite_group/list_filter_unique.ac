from list import List, filter_contained_by_and, unique_implies_tail_unique, unique_length

lemma unique_cons_not_contains[T](head: T, tail: List[T]) {
    List.cons(head, tail).is_unique implies not tail.contains(head)
} by {
    if List.cons(head, tail).is_unique {
        if tail.contains(head) {
            let cons_list = List.cons(head, tail)
            List.cons(head, tail).unique = tail.unique
            tail.unique = List.cons(head, tail)
            unique_length(tail)
            List.cons(head, tail).length = tail.length.suc
            tail.length.suc <= tail.length
            false
        }
    }
}

lemma cons_unique_of_tail_unique_not_contains[T](head: T, tail: List[T]) {
    tail.is_unique and not tail.contains(head) implies List.cons(head, tail).is_unique
} by {
    if tail.is_unique and not tail.contains(head) {
        tail.unique = tail
        List.cons(head, tail).is_unique
    }
}

/// Filtering a unique list preserves uniqueness.
theorem filter_preserves_unique[T](list: List[T], f: T -> Bool) {
    list.is_unique implies list.filter(f).is_unique
} by {
    define p(xs: List[T]) -> Bool {
        xs.is_unique implies xs.filter(f).is_unique
    }

    p(List.nil[T])
    forall(head: T, tail: List[T]) {
        if p(tail) {
            if List.cons(head, tail).is_unique {
                unique_implies_tail_unique(head, tail)
                unique_cons_not_contains(head, tail)
                tail.is_unique
                if f(head) {
                    if tail.filter(f).contains(head) {
                        filter_contained_by_and(tail, f, head)
                        false
                    }
                    cons_unique_of_tail_unique_not_contains(head, tail.filter(f))
                    List.cons(head, tail).filter(f).is_unique
                }
                if not f(head) {
                    let cons_list = List.cons(head, tail)
                    List.cons(head, tail).filter(f).is_unique
                }
                List.cons(head, tail).filter(f).is_unique
            }
            if not List.cons(head, tail).is_unique {
            }
            p(List.cons(head, tail))
        }
    }

    List.induction(function(xs: List[T]) { p(xs) })
    forall(xs: List[T]) {
        p(xs)
    }
}
