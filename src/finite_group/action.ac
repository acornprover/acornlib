from finite_group.base import FiniteGroup, FiniteSubgroup, finite_subgroup_contains_eq,
    finite_subgroup_as_set_cardinality_is_order
from algebra.subgroup import subgroup_constraint
from algebra.group_action import MulAction, orbit, orbit_contains_action, orbit_contains_witness,
    stabilizer, stabilizer_contains_eq, orbit_action_eq_iff_stabilizer_left_coset_raw
from list import List, contains_imp_unique_contains, filter_equivalent_to_and, map,
    map_contains, map_contains_of_contains, unique_contains_imp_contains,
    unique_list_is_unique
from data.basic.set import list_set, list_set_contains_eq, set_ext, unique_list_set_cardinality_is_length
from finite_group.list_filter_unique import filter_preserves_unique

/// The stabilizer of a point in a finite group action, bundled as a finite subgroup.
let finite_stabilizer[G: FiniteGroup, X](a: MulAction[G, X], x: X) -> result: FiniteSubgroup[G] satisfy {
    FiniteSubgroup.new(G.elements.filter(stabilizer(a, x).contains)) = Option.some(result)
} by {
    filter_preserves_unique(G.elements, stabilizer(a, x).contains)
    forall(g: G) {
        filter_equivalent_to_and(G.elements, stabilizer(a, x).contains, g)
        G.elements.filter(stabilizer(a, x).contains).contains(g) = stabilizer(a, x).contains(g)
    }
    subgroup_constraint(G.elements.filter(stabilizer(a, x).contains).contains)
    FiniteSubgroup.constraint(G.elements.filter(stabilizer(a, x).contains))
}

/// Membership in the finite stabilizer is the usual stabilizer membership.
theorem finite_stabilizer_contains_eq[G: FiniteGroup, X](
    a: MulAction[G, X],
    x: X,
    g: G
) {
    finite_stabilizer(a, x).contains(g) = stabilizer(a, x).contains(g)
} by {
    finite_subgroup_contains_eq(finite_stabilizer(a, x), g)
    finite_stabilizer(a, x).elements = G.elements.filter(stabilizer(a, x).contains)
    filter_equivalent_to_and(G.elements, stabilizer(a, x).contains, g)
    G.elements.contains_every
    G.elements.contains(g)
}

/// Membership in the finite stabilizer is exactly fixing the point.
theorem finite_stabilizer_fixes_eq[G: FiniteGroup, X](
    a: MulAction[G, X],
    x: X,
    g: G
) {
    finite_stabilizer(a, x).contains(g) = (a.act(g, x) = x)
} by {
    finite_stabilizer_contains_eq(a, x, g)
    stabilizer_contains_eq(a, x, g)
}

/// The underlying set of the finite stabilizer has cardinality its order.
theorem finite_stabilizer_as_set_cardinality_is_order[G: FiniteGroup, X](
    a: MulAction[G, X],
    x: X
) {
    finite_stabilizer(a, x).as_set.cardinality_is(finite_stabilizer(a, x).order)
} by {
    finite_subgroup_as_set_cardinality_is_order(finite_stabilizer(a, x))
}

/// Two group elements act equally on a point exactly when their quotient lies in the finite stabilizer.
theorem orbit_action_eq_iff_finite_stabilizer_left_coset[G: FiniteGroup, X](
    a: MulAction[G, X],
    x: X,
    g: G,
    k: G
) {
    (a.act(g, x) = a.act(k, x)) = finite_stabilizer(a, x).contains(g.inverse * k)
} by {
    orbit_action_eq_iff_stabilizer_left_coset_raw(a, x, g, k)
    finite_stabilizer_contains_eq(a, x, g.inverse * k)
}

/// The orbit of a point in a finite group action, listed without duplicate values.
define finite_orbit_list[G: FiniteGroup, X](a: MulAction[G, X], x: X) -> List[X] {
    map[G, X](G.elements, function(g: G) { a.act(g, x) }).unique
}

/// The finite orbit list represents exactly the orbit of the point.
theorem finite_orbit_list_set_eq_orbit[G: FiniteGroup, X](a: MulAction[G, X], x: X) {
    list_set(finite_orbit_list(a, x)) = orbit(a, x)
} by {
    let orbit_items = map[G, X](G.elements, function(g: G) { a.act(g, x) })
    forall(y: X) {
        if list_set(finite_orbit_list(a, x)).contains(y) {
            list_set_contains_eq(finite_orbit_list(a, x), y)
            orbit_items.unique.contains(y)
            unique_contains_imp_contains(orbit_items, y)
            map_contains(G.elements, function(g: G) { a.act(g, x) }, y)
            let g: G satisfy {
                G.elements.contains(g) and a.act(g, x) = y
            }
            orbit_contains_action(a, x, g)
            orbit(a, x).contains(y)
        }
        if orbit(a, x).contains(y) {
            orbit_contains_witness(a, x, y)
            let g: G satisfy {
                y = a.act(g, x)
            }
            G.elements.contains(g)
            map_contains_of_contains(G.elements, function(h: G) { a.act(h, x) }, g)
            contains_imp_unique_contains(orbit_items, y)
            orbit_items.unique.contains(y)
            list_set_contains_eq(finite_orbit_list(a, x), y)
            list_set(finite_orbit_list(a, x)).contains(y)
        }
        list_set(finite_orbit_list(a, x)).contains(y) = orbit(a, x).contains(y)
    }
    set_ext(list_set(finite_orbit_list(a, x)), orbit(a, x))
}

/// The orbit of a point in a finite group action has cardinality the finite orbit-list length.
theorem finite_orbit_cardinality_is_length[G: FiniteGroup, X](a: MulAction[G, X], x: X) {
    orbit(a, x).cardinality_is(finite_orbit_list(a, x).length)
} by {
    let orbit_items = map[G, X](G.elements, function(g: G) { a.act(g, x) })
    unique_list_is_unique(orbit_items)
    unique_list_set_cardinality_is_length(finite_orbit_list(a, x))
    list_set(finite_orbit_list(a, x)).cardinality_is(finite_orbit_list(a, x).length)
    finite_orbit_list_set_eq_orbit(a, x)
}
