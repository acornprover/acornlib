from finite_group.base import FiniteGroup, FiniteSubgroup, finite_subgroup_as_set_cardinality_is_order
from algebra.group import Group, left_cancel
from data.basic.functions import is_injective_fn
from data.basic.set import set_image, set_image_cardinality_is_of_injective

lemma left_mul_is_injective_fn[G: Group](a: G) {
    is_injective_fn[G, G](function(h: G) { a * h })
} by {
    forall(x: G, y: G) {
        if (function(h: G) { a * h })(x) = (function(h: G) { a * h })(y) {
            left_cancel[G](a, x, y)
        }
    }
}

/// Left multiplication sends a finite subgroup to an image with cardinality equal to the subgroup order.
theorem finite_subgroup_left_mul_image_cardinality_is_order[G: FiniteGroup](s: FiniteSubgroup[G], a: G) {
    set_image[G, G](s.as_set, function(h: G) { a * h }).cardinality_is(s.order)
} by {
    finite_subgroup_as_set_cardinality_is_order[G](s)
    left_mul_is_injective_fn[G](a)
    set_image_cardinality_is_of_injective[G, G](s.as_set, function(h: G) { a * h }, s.order)
}
