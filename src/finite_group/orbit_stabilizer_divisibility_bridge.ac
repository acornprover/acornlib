from finite_group.action import finite_stabilizer, finite_orbit_list
from finite_group.action_lagrange import finite_subgroup_order_divides_group_order
from finite_group.base import FiniteGroup
from finite_group.orbit_stabilizer import finite_stabilizer_order_mul_finite_orbit_list_length_eq_group_order
from algebra.group_action import MulAction
from nat import Nat

/// The length of a finite orbit list divides the order of the finite group.
theorem finite_orbit_list_length_divides_group_order[G: FiniteGroup, X](a: MulAction[G, X], x: X) {
    finite_orbit_list[G, X](a, x).length.divides(G.order)
} by {
    let orbit_len: Nat = finite_orbit_list[G, X](a, x).length
    let stabilizer_order: Nat = finite_stabilizer[G, X](a, x).order
    finite_stabilizer_order_mul_finite_orbit_list_length_eq_group_order[G, X](a, x)
    stabilizer_order * orbit_len = G.order
    orbit_len * stabilizer_order = stabilizer_order * orbit_len
    orbit_len * stabilizer_order = G.order
    finite_orbit_list[G, X](a, x).length.divides(G.order)
}

/// The stabilizer order attached to a finite group action divides the group order.
theorem finite_stabilizer_order_divides_group_order_from_action[G: FiniteGroup, X](a: MulAction[G, X], x: X) {
    finite_stabilizer[G, X](a, x).order.divides(G.order)
} by {
    finite_subgroup_order_divides_group_order[G](finite_stabilizer[G, X](a, x))
}

/// The orbit length and stabilizer order both divide the finite group order.
theorem finite_orbit_stabilizer_divides_pair[G: FiniteGroup, X](a: MulAction[G, X], x: X) {
    finite_orbit_list[G, X](a, x).length.divides(G.order) and
    finite_stabilizer[G, X](a, x).order.divides(G.order)
} by {
    finite_orbit_list_length_divides_group_order[G, X](a, x)
    finite_stabilizer_order_divides_group_order_from_action[G, X](a, x)
}
