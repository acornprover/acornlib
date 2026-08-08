from finite_group.base import FiniteGroup, FiniteSubgroup,
    finite_subgroup_as_set_contains_finite_subgroup_eq,
    finite_subgroup_contains_as_subgroup_eq
from nat import Nat
from algebra.subgroup import subgroup_contains_identity, subgroup_inv_mem, subgroup_mul_inv_mem,
    subgroup_mul_mem, subgroup_pow_mem

/// A finite subgroup contains the identity element of its ambient finite group.
theorem finite_subgroup_contains_identity_direct[G: FiniteGroup](s: FiniteSubgroup[G]) {
    s.contains(G.1)
} by {
    subgroup_contains_identity(s.as_subgroup)
    s.as_subgroup.contains(G.1)
    finite_subgroup_contains_as_subgroup_eq(s, G.1)
    s.contains(G.1)
}

/// A finite subgroup is closed under multiplication of its members.
theorem finite_subgroup_mul_mem_direct[G: FiniteGroup](s: FiniteSubgroup[G], a: G, b: G) {
    s.contains(a) and s.contains(b) implies s.contains(a * b)
} by {
    if s.contains(a) and s.contains(b) {
        finite_subgroup_contains_as_subgroup_eq(s, a)
        finite_subgroup_contains_as_subgroup_eq(s, b)
        s.as_subgroup.contains(a)
        s.as_subgroup.contains(b)
        subgroup_mul_mem(s.as_subgroup, a, b)
        s.as_subgroup.contains(a * b)
        finite_subgroup_contains_as_subgroup_eq(s, a * b)
        s.contains(a * b)
    }
}

/// A finite subgroup is closed under inverses of its members.
theorem finite_subgroup_inv_mem_direct[G: FiniteGroup](s: FiniteSubgroup[G], a: G) {
    s.contains(a) implies s.contains(a.inverse)
} by {
    if s.contains(a) {
        finite_subgroup_contains_as_subgroup_eq(s, a)
        s.as_subgroup.contains(a)
        subgroup_inv_mem(s.as_subgroup, a)
        s.as_subgroup.contains(a.inverse)
        finite_subgroup_contains_as_subgroup_eq(s, a.inverse)
        s.contains(a.inverse)
    }
}

/// A finite subgroup is closed under multiplying a member by the inverse of another member.
theorem finite_subgroup_mul_inv_mem_direct[G: FiniteGroup](s: FiniteSubgroup[G], a: G, b: G) {
    s.contains(a) and s.contains(b) implies s.contains(a * b.inverse)
} by {
    if s.contains(a) and s.contains(b) {
        finite_subgroup_contains_as_subgroup_eq(s, a)
        finite_subgroup_contains_as_subgroup_eq(s, b)
        s.as_subgroup.contains(a)
        s.as_subgroup.contains(b)
        subgroup_mul_inv_mem(s.as_subgroup, a, b)
        s.as_subgroup.contains(a * b.inverse)
        finite_subgroup_contains_as_subgroup_eq(s, a * b.inverse)
        s.contains(a * b.inverse)
    }
}

/// A finite subgroup contains every natural power of each of its members.
theorem finite_subgroup_pow_mem_direct[G: FiniteGroup](s: FiniteSubgroup[G], a: G, n: Nat) {
    s.contains(a) implies s.contains(a.pow(n))
} by {
    if s.contains(a) {
        finite_subgroup_contains_as_subgroup_eq(s, a)
        s.as_subgroup.contains(a)
        subgroup_pow_mem(s.as_subgroup, a, n)
        s.as_subgroup.contains(a.pow(n))
        finite_subgroup_contains_as_subgroup_eq(s, a.pow(n))
        s.contains(a.pow(n))
    }
}

/// The underlying set of a finite subgroup is closed under multiplication.
theorem finite_subgroup_as_set_closed_mul[G: FiniteGroup](s: FiniteSubgroup[G], a: G, b: G) {
    s.as_set.contains(a) and s.as_set.contains(b) implies s.as_set.contains(a * b)
} by {
    if s.as_set.contains(a) and s.as_set.contains(b) {
        finite_subgroup_as_set_contains_finite_subgroup_eq(s, a)
        finite_subgroup_as_set_contains_finite_subgroup_eq(s, b)
        s.contains(a)
        s.contains(b)
        finite_subgroup_mul_mem_direct(s, a, b)
        s.contains(a * b)
        finite_subgroup_as_set_contains_finite_subgroup_eq(s, a * b)
        s.as_set.contains(a * b)
    }
}

/// The underlying set of a finite subgroup is closed under inverses.
theorem finite_subgroup_as_set_closed_inv[G: FiniteGroup](s: FiniteSubgroup[G], a: G) {
    s.as_set.contains(a) implies s.as_set.contains(a.inverse)
} by {
    if s.as_set.contains(a) {
        finite_subgroup_as_set_contains_finite_subgroup_eq(s, a)
        s.contains(a)
        finite_subgroup_inv_mem_direct(s, a)
        s.contains(a.inverse)
        finite_subgroup_as_set_contains_finite_subgroup_eq(s, a.inverse)
        s.as_set.contains(a.inverse)
    }
}
