from algebra.group import Group
from algebra.subgroup import subgroup_constraint, identity_constraint, closure_constraint, inverse_constraint
from finite_group.base import FiniteGroup, FiniteSubgroup, finite_subgroup_contains_eq,
    finite_subgroup_as_set_cardinality_is_order
from finite_group.list_filter_unique import filter_preserves_unique
from list import filter_equivalent_to_and
from data.basic.set import Set, set_ext, singleton_contains_eq
from algebra.group_action import orbit, orbit_contains_self, orbit_contains_action
from algebra.group.conjugation_action import conjugation_act, conjugation_action, conjugation_action_act,
    conjugation_act_fixed_iff_commutes, conjugation_act_inverse,
    conjugation_action_orbit_contains_iff

/// True when `x` commutes with every element of the group.
define is_central_element[G: Group](x: G) -> Bool {
    forall(g: G) {
        g * x = x * g
    }
}

/// The identity element is central.
theorem is_central_element_one[G: Group] {
    is_central_element[G](G.1)
} by {
    forall(g: G) {
        g * G.1 = g
        G.1 * g = g
        g * G.1 = G.1 * g
    }
}

/// The product of two central elements is central.
theorem is_central_element_mul[G: Group](x: G, y: G) {
    is_central_element(x) and is_central_element(y) implies is_central_element(x * y)
} by {
    if is_central_element(x) and is_central_element(y) {
        is_central_element(x) = forall(h: G) {
            h * x = x * h
        }
        is_central_element(y) = forall(h: G) {
            h * y = y * h
        }
        forall(g: G) {
            g * x = x * g
            g * y = y * g
            g * (x * y) = (g * x) * y
            (g * x) * y = (x * g) * y
            (x * g) * y = x * (g * y)
            x * (g * y) = x * (y * g)
            x * (y * g) = (x * y) * g
            g * (x * y) = (x * y) * g
        }
    }
}

/// The inverse of a central element is central.
theorem is_central_element_inv[G: Group](x: G) {
    is_central_element(x) implies is_central_element(x.inverse)
} by {
    if is_central_element(x) {
        is_central_element(x) = forall(h: G) {
            h * x = x * h
        }
        forall(g: G) {
            g * x = x * g
            conjugation_act_fixed_iff_commutes(g, x)
            conjugation_act(g, x) = x
            conjugation_act_inverse(g, x)
            conjugation_act(g, x.inverse) = conjugation_act(g, x).inverse
            conjugation_act(g, x.inverse) = x.inverse
            conjugation_act_fixed_iff_commutes(g, x.inverse)
            g * x.inverse = x.inverse * g
        }
    }
}

/// The central elements satisfy the subgroup constraints.
theorem is_central_element_subgroup_constraint[G: Group] {
    subgroup_constraint(is_central_element[G])
} by {
    is_central_element_one[G]
    identity_constraint(is_central_element[G])
    forall(x: G, y: G) {
        if is_central_element(x) and is_central_element(y) {
            is_central_element_mul(x, y)
        }
    }
    closure_constraint(is_central_element[G])
    forall(x: G) {
        if is_central_element(x) {
            is_central_element_inv(x)
        }
    }
    inverse_constraint(is_central_element[G])
}

/// The filtered list of central elements satisfies the finite-subgroup constraint.
lemma finite_center_elements_constraint[G: FiniteGroup] {
    FiniteSubgroup.constraint(G.elements.filter(is_central_element[G]))
} by {
    filter_preserves_unique(G.elements, is_central_element[G])
    G.elements.filter(is_central_element[G]).is_unique
    forall(g: G) {
        filter_equivalent_to_and(G.elements, is_central_element[G], g)
        G.elements.contains_every
        G.elements.contains(g)
        G.elements.filter(is_central_element[G]).contains(g) = is_central_element(g)
    }
    G.elements.filter(is_central_element[G]).contains = is_central_element[G]
    is_central_element_subgroup_constraint[G]
    subgroup_constraint(G.elements.filter(is_central_element[G]).contains)
    FiniteSubgroup.constraint(G.elements.filter(is_central_element[G]))
}

/// The center of a finite group, bundled as a finite subgroup.
let finite_center[G: FiniteGroup]: FiniteSubgroup[G] satisfy {
    FiniteSubgroup.new(G.elements.filter(is_central_element[G])) = Option.some(finite_center)
}

/// Membership in the finite center is centrality.
theorem finite_center_contains_eq[G: FiniteGroup](x: G) {
    finite_center[G].contains(x) = is_central_element(x)
} by {
    finite_subgroup_contains_eq(finite_center[G], x)
    finite_center[G].elements = G.elements.filter(is_central_element[G])
    filter_equivalent_to_and(G.elements, is_central_element[G], x)
    G.elements.contains_every
    G.elements.contains(x)
}

/// The underlying set of the finite center has cardinality its order.
theorem finite_center_as_set_cardinality_is_order[G: FiniteGroup] {
    finite_center[G].as_set.cardinality_is(finite_center[G].order)
} by {
    finite_subgroup_as_set_cardinality_is_order(finite_center[G])
}

/// A group element is central exactly when every conjugation fixes it.
theorem is_central_element_iff_conjugation_fixed[G: Group](x: G) {
    is_central_element(x) iff forall(g: G) {
        conjugation_action[G].act(g, x) = x
    }
} by {
    if is_central_element(x) {
        is_central_element(x) = forall(h: G) {
            h * x = x * h
        }
        forall(g: G) {
            g * x = x * g
            conjugation_act_fixed_iff_commutes(g, x)
            conjugation_action_act(g, x)
            conjugation_action[G].act(g, x) = x
        }
    }
    if forall(g: G) { conjugation_action[G].act(g, x) = x } {
        forall(g: G) {
            conjugation_action[G].act(g, x) = x
            conjugation_action_act(g, x)
            g * x * g.inverse = x
            conjugation_act_fixed_iff_commutes(g, x)
            g * x = x * g
        }
    }
}

/// For a central element, orbit membership is singleton membership.
lemma central_element_conjugation_orbit_contains_eq_singleton[G: Group](x: G, y: G) {
    is_central_element(x) implies
        orbit(conjugation_action[G], x).contains(y) = (Set[G].singleton(x)).contains(y)
} by {
    if is_central_element(x) {
        is_central_element(x) = forall(h: G) {
            h * x = x * h
        }
        if orbit(conjugation_action[G], x).contains(y) {
            conjugation_action_orbit_contains_iff(x, y)
            let g: G satisfy {
                g * x * g.inverse = y
            }
            g * x = x * g
            g * x * g.inverse = x * g * g.inverse
            x * g * g.inverse = x * (g * g.inverse)
            g * g.inverse = G.1
            x * (g * g.inverse) = x * G.1
            x * G.1 = x
            g * x * g.inverse = x
            y = x
            singleton_contains_eq[G](x, y)
            (Set[G].singleton(x)).contains(y)
        }
        if (Set[G].singleton(x)).contains(y) {
            singleton_contains_eq[G](x, y)
            x = y
            orbit_contains_self(conjugation_action[G], x)
            orbit(conjugation_action[G], x).contains(x)
            orbit(conjugation_action[G], x).contains(y)
        }
        orbit(conjugation_action[G], x).contains(y) = (Set[G].singleton(x)).contains(y)
    }
}

/// A central element has singleton conjugation orbit.
lemma is_central_element_imp_conjugation_orbit_singleton[G: Group](x: G) {
    is_central_element(x) implies orbit(conjugation_action[G], x) = Set[G].singleton(x)
} by {
    if is_central_element(x) {
        let o = orbit(conjugation_action[G], x)
        let s = Set[G].singleton(x)
        forall(y: G) {
            central_element_conjugation_orbit_contains_eq_singleton(x, y)
            is_central_element(x) implies o.contains(y) = s.contains(y)
            o.contains(y) = s.contains(y)
        }
        set_ext(o, s)
        o = s
        orbit(conjugation_action[G], x) = Set[G].singleton(x)
    }
}

/// A singleton conjugation orbit gives fixed conjugation by every group element.
lemma conjugation_orbit_singleton_imp_conjugation_fixed[G: Group](x: G) {
    orbit(conjugation_action[G], x) = Set[G].singleton(x) implies forall(g: G) {
        conjugation_action[G].act(g, x) = x
    }
} by {
    if orbit(conjugation_action[G], x) = Set[G].singleton(x) {
        forall(g: G) {
            orbit_contains_action(conjugation_action[G], x, g)
            orbit(conjugation_action[G], x).contains(conjugation_action[G].act(g, x))
            (Set[G].singleton(x)).contains(conjugation_action[G].act(g, x))
            singleton_contains_eq[G](x, conjugation_action[G].act(g, x))
            x = conjugation_action[G].act(g, x)
            conjugation_action[G].act(g, x) = x
        }
    }
}

/// A singleton conjugation orbit forces centrality.
lemma conjugation_orbit_singleton_imp_is_central_element[G: Group](x: G) {
    orbit(conjugation_action[G], x) = Set[G].singleton(x) implies is_central_element(x)
} by {
    if orbit(conjugation_action[G], x) = Set[G].singleton(x) {
        conjugation_orbit_singleton_imp_conjugation_fixed(x)
        is_central_element_iff_conjugation_fixed(x)
    }
}

/// A group element is central exactly when its conjugation orbit is the singleton containing it.
theorem is_central_element_iff_conjugation_orbit_singleton[G: Group](x: G) {
    is_central_element(x) iff orbit(conjugation_action[G], x) = Set[G].singleton(x)
} by {
    is_central_element_imp_conjugation_orbit_singleton(x)
    conjugation_orbit_singleton_imp_is_central_element(x)
}

/// Membership in the finite center is equivalent to having a singleton conjugation orbit.
theorem finite_center_contains_iff_conjugation_orbit_singleton[G: FiniteGroup](x: G) {
    finite_center[G].contains(x) iff orbit(conjugation_action[G], x) = Set[G].singleton(x)
} by {
    finite_center_contains_eq(x)
    is_central_element_iff_conjugation_orbit_singleton(x)
}
