from finite_group.base import FiniteGroup, cyclic_subgroup_of,
    cyclic_subgroup_order_pos, pow_cyclic_subgroup_order_eq_identity,
    pow_mod_cyclic_subgroup_order, pow_add_cyclic_subgroup_order,
    cyclic_subgroup_contains_power, cyclic_subgroup_contains_generator,
    cyclic_subgroup_as_set_contains_generator,
    cyclic_subgroup_as_finite_set_contains_generator,
    cyclic_subgroup_order_le_period, no_positive_period_below_cyclic_subgroup_order
from nat import Nat
numerals Nat

/// The cyclic subgroup order is a positive period, and powers repeat after adding it.
theorem cyclic_order_periodic_power[G: FiniteGroup](g: G, k: Nat) {
    cyclic_subgroup_of(g).order > Nat.0 and
    g.pow(cyclic_subgroup_of(g).order) = G.1 and
    g.pow(k + cyclic_subgroup_of(g).order) = g.pow(k)
} by {
    cyclic_subgroup_order_pos(g)
    pow_cyclic_subgroup_order_eq_identity(g)
    pow_add_cyclic_subgroup_order(g, k)
}

/// Reducing an exponent modulo the cyclic subgroup order preserves the power and stays in the cyclic subgroup.
theorem cyclic_order_mod_period_power[G: FiniteGroup](g: G, k: Nat) {
    cyclic_subgroup_of(g).order > Nat.0 and
    g.pow(k.mod(cyclic_subgroup_of(g).order)) = g.pow(k) and
    cyclic_subgroup_of(g).contains(g.pow(k.mod(cyclic_subgroup_of(g).order))) and
    cyclic_subgroup_of(g).contains(g.pow(k))
} by {
    cyclic_subgroup_order_pos(g)
    pow_mod_cyclic_subgroup_order(g, k)
    cyclic_subgroup_contains_power(g, k.mod(cyclic_subgroup_of(g).order))
    cyclic_subgroup_contains_power(g, k)
}

/// The generator occurs as the first positive power and belongs to each bundled cyclic carrier.
theorem cyclic_order_generator_power_one[G: FiniteGroup](g: G) {
    g.pow(Nat.1) = g and
    cyclic_subgroup_of(g).contains(g) and
    cyclic_subgroup_of(g).as_set.contains(g) and
    cyclic_subgroup_of(g).as_finite_set.contains(g)
} by {
    g.pow(Nat.1) = g
    cyclic_subgroup_contains_generator(g)
    cyclic_subgroup_as_set_contains_generator(g)
    cyclic_subgroup_as_finite_set_contains_generator(g)
}

/// The cyclic subgroup order is a positive identity power, and that identity lies in the generated subgroup.
theorem cyclic_order_identity_power[G: FiniteGroup](g: G) {
    cyclic_subgroup_of(g).order > Nat.0 and
    g.pow(cyclic_subgroup_of(g).order) = G.1 and
    cyclic_subgroup_of(g).contains(G.1)
} by {
    cyclic_subgroup_order_pos(g)
    pow_cyclic_subgroup_order_eq_identity(g)
    cyclic_subgroup_contains_power(g, cyclic_subgroup_of(g).order)
}

/// A positive identity power cannot be smaller than the cyclic subgroup order.
theorem cyclic_order_no_smaller_positive_period[G: FiniteGroup](g: G, k: Nat) {
    k > Nat.0 and g.pow(k) = G.1 implies
        cyclic_subgroup_of(g).order <= k and not k < cyclic_subgroup_of(g).order
} by {
    if k > Nat.0 and g.pow(k) = G.1 {
        cyclic_subgroup_order_le_period(g, k)
        cyclic_subgroup_of(g).order <= k
        if k < cyclic_subgroup_of(g).order {
            no_positive_period_below_cyclic_subgroup_order(g, k)
            g.pow(k) != G.1
            false
        }
    }
}
