from finite_group.base import FiniteGroup
from finite_group.action import finite_stabilizer, finite_orbit_list
from finite_group.action_counting import finite_left_coset_set, finite_left_coset_representatives,
    finite_orbit_action_eq_iff_finite_stabilizer_left_coset_set_eq
from finite_group.action_lagrange import finite_left_coset_representatives_order_product_eq_group_order
from algebra.group_action import MulAction
from list import List, map, map_contains, map_contains_of_contains
from nat import Nat
from data.basic.set import Set

numerals Nat

lemma list_induction_elim_local[T](p: List[T] -> Bool, items: List[T]) {
    p(List.nil[T]) and forall(head: T, tail: List[T]) { p(tail) implies p(List.cons[T](head, tail)) }
    implies p(items)
} by {
    if p(List.nil[T]) and forall(head: T, tail: List[T]) { p(tail) implies p(List.cons[T](head, tail)) } {
        List.induction(p)
        forall(xs: List[T]) { p(xs) }
        p(items)
    }
}

lemma same_kernel_map_tail_contains_head_forward[T, U, V](
    tail: List[T],
    f: T -> U,
    h: T -> V,
    head: T
) {
    (forall(x: T, y: T) { (f(x) = f(y)) = (h(x) = h(y)) }) and
    map[T, U](tail, f).contains(f(head)) implies map[T, V](tail, h).contains(h(head))
} by {
    if forall(x: T, y: T) { (f(x) = f(y)) = (h(x) = h(y)) } and
        map[T, U](tail, f).contains(f(head)) {
        map_contains[T, U](tail, f, f(head))
        let x: T satisfy {
            tail.contains(x) and f(x) = f(head)
        }
        h(x) = h(head)
        map_contains_of_contains[T, V](tail, h, x)
        map[T, V](tail, h).contains(h(head))
    }
}

lemma same_kernel_map_tail_contains_head_reverse[T, U, V](
    tail: List[T],
    f: T -> U,
    h: T -> V,
    head: T
) {
    (forall(x: T, y: T) { (f(x) = f(y)) = (h(x) = h(y)) }) and
    map[T, V](tail, h).contains(h(head)) implies map[T, U](tail, f).contains(f(head))
} by {
    if forall(x: T, y: T) { (f(x) = f(y)) = (h(x) = h(y)) } and
        map[T, V](tail, h).contains(h(head)) {
        map_contains[T, V](tail, h, h(head))
        let x: T satisfy {
            tail.contains(x) and h(x) = h(head)
        }
        f(x) = f(head)
        map_contains_of_contains[T, U](tail, f, x)
        map[T, U](tail, f).contains(f(head))
    }
}

lemma same_kernel_unique_map_length_nil[T, U, V](f: T -> U, h: T -> V) {
    map[T, U](List.nil[T], f).unique.length = map[T, V](List.nil[T], h).unique.length
} by {
    List.nil[U].unique.length = Nat.0
    List.nil[V].unique.length = Nat.0
}

lemma same_kernel_unique_map_length_cons_step[T, U, V](
    tail: List[T],
    f: T -> U,
    h: T -> V,
    head: T
) {
    (forall(x: T, y: T) { (f(x) = f(y)) = (h(x) = h(y)) }) and
    map[T, U](tail, f).unique.length = map[T, V](tail, h).unique.length
    implies map[T, U](List.cons[T](head, tail), f).unique.length =
        map[T, V](List.cons[T](head, tail), h).unique.length
} by {
    if forall(x: T, y: T) { (f(x) = f(y)) = (h(x) = h(y)) } and
        map[T, U](tail, f).unique.length = map[T, V](tail, h).unique.length {
        if map[T, U](tail, f).contains(f(head)) {
            same_kernel_map_tail_contains_head_forward[T, U, V](tail, f, h, head)
            map[T, U](List.cons[T](head, tail), f).unique.length = map[T, U](tail, f).unique.length
            map[T, V](List.cons[T](head, tail), h).unique.length = map[T, V](tail, h).unique.length
            map[T, U](List.cons[T](head, tail), f).unique.length =
                map[T, V](List.cons[T](head, tail), h).unique.length
        } else {
            if map[T, V](tail, h).contains(h(head)) {
                same_kernel_map_tail_contains_head_reverse[T, U, V](tail, f, h, head)
                false
            }
            List.cons[U](f(head), map[T, U](tail, f)).unique =
                List.cons[U](f(head), map[T, U](tail, f).unique)
            List.cons[V](h(head), map[T, V](tail, h)).unique =
                List.cons[V](h(head), map[T, V](tail, h).unique)
            map[T, U](List.cons[T](head, tail), f).unique.length =
                map[T, U](tail, f).unique.length.suc
            map[T, V](List.cons[T](head, tail), h).unique.length =
                map[T, V](tail, h).unique.length.suc
            map[T, U](List.cons[T](head, tail), f).unique.length =
                map[T, V](List.cons[T](head, tail), h).unique.length
        }
    }
}

lemma same_kernel_unique_map_length_eq[T, U, V](items: List[T], f: T -> U, h: T -> V) {
    (forall(x: T, y: T) { (f(x) = f(y)) = (h(x) = h(y)) }) implies
    map[T, U](items, f).unique.length = map[T, V](items, h).unique.length
} by {
    if forall(x: T, y: T) { (f(x) = f(y)) = (h(x) = h(y)) } {
        let p: List[T] -> Bool = function(xs: List[T]) {
            map[T, U](xs, f).unique.length = map[T, V](xs, h).unique.length
        }
        same_kernel_unique_map_length_nil[T, U, V](f, h)
        p(List.nil[T])
        forall(head: T, tail: List[T]) {
            if p(tail) {
                map[T, U](tail, f).unique.length = map[T, V](tail, h).unique.length
                same_kernel_unique_map_length_cons_step[T, U, V](tail, f, h, head)
                p(List.cons[T](head, tail))
            }
        }
        list_induction_elim_local[T](p, items)
        p(items) = (map[T, U](items, f).unique.length = map[T, V](items, h).unique.length)
        map[T, U](items, f).unique.length = map[T, V](items, h).unique.length
    }
}

/// The finite orbit list and finite-stabilizer left-coset representatives have the same length.
theorem finite_orbit_list_length_eq_finite_stabilizer_cosets[G: FiniteGroup, X](
    a: MulAction[G, X],
    x: X
) {
    finite_orbit_list[G, X](a, x).length =
        finite_left_coset_representatives[G](finite_stabilizer[G, X](a, x)).length
} by {
    forall(g: G, k: G) {
        finite_orbit_action_eq_iff_finite_stabilizer_left_coset_set_eq[G, X](a, x, g, k)
        (function(y: G) { a.act(y, x) })(g) = (function(y: G) { a.act(y, x) })(k) =
            (a.act(g, x) = a.act(k, x))
        (function(y: G) { finite_left_coset_set[G](finite_stabilizer[G, X](a, x), y) })(g) =
            (function(y: G) { finite_left_coset_set[G](finite_stabilizer[G, X](a, x), y) })(k) =
            (finite_left_coset_set[G](finite_stabilizer[G, X](a, x), g) =
                finite_left_coset_set[G](finite_stabilizer[G, X](a, x), k))
    }
    same_kernel_unique_map_length_eq[G, X, Set[G]](
        G.elements,
        function(g: G) { a.act(g, x) },
        function(g: G) { finite_left_coset_set[G](finite_stabilizer[G, X](a, x), g) }
    )
    map[G, X](G.elements, function(g: G) { a.act(g, x) }).unique.length =
        map[G, Set[G]](
            G.elements,
            function(g: G) { finite_left_coset_set[G](finite_stabilizer[G, X](a, x), g) }
        ).unique.length
}

/// Orbit-stabilizer product formula for the finite orbit list.
theorem finite_stabilizer_order_mul_finite_orbit_list_length_eq_group_order[G: FiniteGroup, X](
    a: MulAction[G, X],
    x: X
) {
    finite_stabilizer[G, X](a, x).order * finite_orbit_list[G, X](a, x).length = G.order
} by {
    finite_orbit_list_length_eq_finite_stabilizer_cosets[G, X](a, x)
    finite_left_coset_representatives_order_product_eq_group_order[G](finite_stabilizer[G, X](a, x))
}
