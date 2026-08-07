from finite_group.base import FiniteGroup, FiniteSubgroup
from finite_group.action_counting import finite_left_coset_representatives
from finite_group.action_lagrange import finite_left_coset_representatives_is_unique,
    finite_left_coset_representatives_pairwise_disjoint,
    finite_left_coset_representatives_union_covers_universal,
    finite_left_coset_representatives_all_cardinality_is_order
from data.basic.set import Set
from data.basic.set_list_partition import set_list_exactly_covers, set_list_partition_of,
    set_list_all_cardinality_is

/// Finite left-coset representatives exactly cover the whole finite group.
theorem finite_left_coset_representatives_exactly_cover_universal[G: FiniteGroup](
    s: FiniteSubgroup[G]
) {
    set_list_exactly_covers[G](finite_left_coset_representatives(s), Set[G].universal_set)
} by {
    finite_left_coset_representatives_union_covers_universal(s)
}

/// Finite left-coset representatives form a partition of the whole finite group.
theorem finite_left_coset_representatives_partition_of_universal[G: FiniteGroup](
    s: FiniteSubgroup[G]
) {
    set_list_partition_of[G](finite_left_coset_representatives(s), Set[G].universal_set)
} by {
    finite_left_coset_representatives_exactly_cover_universal(s)
    finite_left_coset_representatives_is_unique(s)
    finite_left_coset_representatives_pairwise_disjoint(s)
}

/// The finite left-coset partition is uniform, with each part having subgroup order.
theorem finite_left_coset_partition_uniform_cardinality[G: FiniteGroup](s: FiniteSubgroup[G]) {
    set_list_partition_of[G](finite_left_coset_representatives(s), Set[G].universal_set) and
    set_list_all_cardinality_is[G](finite_left_coset_representatives(s), s.order)
} by {
    finite_left_coset_representatives_partition_of_universal(s)
    finite_left_coset_representatives_all_cardinality_is_order(s)
}
