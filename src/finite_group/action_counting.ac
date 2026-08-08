from finite_group.base import FiniteGroup, FiniteSubgroup, finite_subgroup_contains_as_subgroup_eq
from finite_group.action import finite_stabilizer, orbit_action_eq_iff_finite_stabilizer_left_coset
from algebra.group import Group
from algebra.semigroup import Semigroup
from algebra.subgroup import Subgroup, subgroup_contains_identity
from algebra.group_action import MulAction
from data.basic.set import Set, set_ext
from list import List, map, map_contains_of_contains

theorem same_left_coset_contains_symmetric[G: Group](s: Subgroup[G], a: G, b: G) {
    s.contains(a.inverse * b) implies s.contains(b.inverse * a)
} by {
}

theorem same_left_coset_contains_transitive[G: Group](s: Subgroup[G], a: G, b: G, c: G) {
    s.contains(a.inverse * b) and s.contains(b.inverse * c) implies s.contains(a.inverse * c)
} by {
    if s.contains(a.inverse * b) and s.contains(b.inverse * c) {
        s.contains((a.inverse * b) * (b.inverse * c))
        Semigroup.mul_associative(a.inverse, b, b.inverse * c)
        Semigroup.mul_associative(b, b.inverse, c)
        s.contains(a.inverse * c)
    }
}

theorem same_left_coset_contains_membership_eq[G: Group](s: Subgroup[G], a: G, b: G, y: G) {
    s.contains(a.inverse * b) implies s.contains(a.inverse * y) = s.contains(b.inverse * y)
} by {
    if s.contains(a.inverse * y) {
        same_left_coset_contains_symmetric(s, a, b)
        same_left_coset_contains_transitive(s, b, a, y)
        s.contains(b.inverse * y)
    }
    if s.contains(b.inverse * y) {
        same_left_coset_contains_transitive(s, a, b, y)
        s.contains(a.inverse * y)
    }
}

/// True when a group element belongs to a finite left coset.
define finite_left_coset_contains[G: FiniteGroup](s: FiniteSubgroup[G], a: G, y: G) -> Bool {
    s.contains(a.inverse * y)
}

/// The left coset of a finite subgroup represented as a finite set predicate.
define finite_left_coset_set[G: FiniteGroup](s: FiniteSubgroup[G], a: G) -> Set[G] {
    Set[G].new(finite_left_coset_contains(s, a))
}

/// Membership in a finite left coset is membership of the quotient in the finite subgroup.
theorem finite_left_coset_set_contains_eq[G: FiniteGroup](s: FiniteSubgroup[G], a: G, y: G) {
    finite_left_coset_set(s, a).contains(y) = s.contains(a.inverse * y)
} by {
    finite_left_coset_contains(s, a, y) = s.contains(a.inverse * y)
}

/// A representative belongs to its finite left coset.
theorem finite_left_coset_set_contains_representative[G: FiniteGroup](s: FiniteSubgroup[G], a: G) {
    finite_left_coset_set(s, a).contains(a)
} by {
    subgroup_contains_identity(s.as_subgroup)
    finite_subgroup_contains_as_subgroup_eq(s, a.inverse * a)
    s.contains(a.inverse * a)
    finite_left_coset_set_contains_eq(s, a, a)
}

/// Equal finite left-coset representatives give equal finite left-coset predicates.
theorem finite_left_coset_set_eq_of_contains_quotient[G: FiniteGroup](s: FiniteSubgroup[G], a: G, b: G) {
    s.contains(a.inverse * b) implies finite_left_coset_set(s, a) = finite_left_coset_set(s, b)
} by {
    if s.contains(a.inverse * b) {
        finite_subgroup_contains_as_subgroup_eq(s, a.inverse * b)
        forall(y: G) {
            same_left_coset_contains_membership_eq(s.as_subgroup, a, b, y)
            finite_subgroup_contains_as_subgroup_eq(s, a.inverse * y)
            finite_subgroup_contains_as_subgroup_eq(s, b.inverse * y)
            finite_left_coset_set(s, a).contains(y) = finite_left_coset_set(s, b).contains(y)
        }
        set_ext(finite_left_coset_set(s, a), finite_left_coset_set(s, b))
        finite_left_coset_set(s, a) = finite_left_coset_set(s, b)
    }
}

/// Equal finite left-coset predicates have representatives in the same finite left coset.
theorem finite_left_coset_set_eq_implies_contains_quotient[G: FiniteGroup](s: FiniteSubgroup[G], a: G, b: G) {
    finite_left_coset_set(s, a) = finite_left_coset_set(s, b) implies s.contains(a.inverse * b)
} by {
    if finite_left_coset_set(s, a) = finite_left_coset_set(s, b) {
        finite_left_coset_set_contains_representative(s, b)
        finite_left_coset_set_contains_eq(s, a, b)
        s.contains(a.inverse * b)
    }
}

/// Two finite left-coset predicates are equal exactly when the representative quotient lies in the subgroup.
theorem finite_left_coset_set_eq_iff_contains_quotient[G: FiniteGroup](s: FiniteSubgroup[G], a: G, b: G) {
    (finite_left_coset_set(s, a) = finite_left_coset_set(s, b)) = s.contains(a.inverse * b)
} by {
    if finite_left_coset_set(s, a) = finite_left_coset_set(s, b) {
        finite_left_coset_set_eq_implies_contains_quotient(s, a, b)
    }
    if s.contains(a.inverse * b) {
        finite_left_coset_set_eq_of_contains_quotient(s, a, b)
    }
}

/// The duplicate-free list of finite left cosets of a finite subgroup.
define finite_left_coset_representatives[G: FiniteGroup](s: FiniteSubgroup[G]) -> List[Set[G]] {
    map[G, Set[G]](G.elements, function(a: G) { finite_left_coset_set(s, a) }).unique
}

/// Every group element contributes its finite left coset to the representative list.
theorem finite_left_coset_representatives_contains_coset[G: FiniteGroup](s: FiniteSubgroup[G], a: G) {
    finite_left_coset_representatives(s).contains(finite_left_coset_set(s, a))
} by {
    map_contains_of_contains[G, Set[G]](G.elements, function(g: G) { finite_left_coset_set(s, g) }, a)
    map[G, Set[G]](G.elements, function(g: G) { finite_left_coset_set(s, g) }).contains(finite_left_coset_set(s, a))
}

/// The orbit map and finite-stabilizer left-coset map have the same equality kernel.
theorem finite_orbit_action_eq_iff_finite_stabilizer_left_coset_set_eq[G: FiniteGroup, X](
    a: MulAction[G, X],
    x: X,
    g: G,
    k: G
) {
    (a.act(g, x) = a.act(k, x)) =
        (finite_left_coset_set(finite_stabilizer(a, x), g) = finite_left_coset_set(finite_stabilizer(a, x), k))
} by {
    orbit_action_eq_iff_finite_stabilizer_left_coset(a, x, g, k)
    finite_left_coset_set_eq_iff_contains_quotient(finite_stabilizer(a, x), g, k)
}
