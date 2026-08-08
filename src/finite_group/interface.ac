from list import List, map
from algebra.group import Group, has_finite_order
from algebra.group_action import MulAction, orbit, stabilizer
from nat import Nat
numerals Nat

/// A group is finite if its elements can be placed in a (finite) list
typeclass G: FiniteGroup extends Group {
    /// A list containing all elements of the group.
    elements: List[G]

    /// Every group element appears in the elements list.
    all_group_elements_in_elements(g: G) {
        G.elements.contains_every
    }
    /// The elements list contains no duplicates.
    unique_elements_list {
        G.elements.is_unique
    }
}

attributes G: FiniteGroup {
    /// The number of elements in the group.
    let order: Nat = G.elements.length
}

from algebra.subgroup import Subgroup, subgroup_constraint, identity_constraint, inverse_constraint, closure_constraint,
    subgroup_is_finitely_generated
from data.basic.set import list_set
from algebra.subsemigroup import Subsemigroup, subsemigroup_is_finitely_generated
from algebra.monoid.submonoid import Submonoid, submonoid_is_finitely_generated
from finite_set import FiniteSet
from data.basic.set import Set, finite_constraint
from data.basic.set_list_partition import set_list_union, set_list_exactly_covers, set_list_partition_of,
    set_list_all_cardinality_is

theorem subsemigroup_of_finite_group_is_finite[G: FiniteGroup](s: Subsemigroup[G]) {
    finite_constraint(s.contains)
}

theorem subsemigroup_as_set_is_finite[G: FiniteGroup](s: Subsemigroup[G]) {
    s.as_set.is_finite
}

theorem subsemigroup_as_set_cardinality_at_most_group_order[G: FiniteGroup](s: Subsemigroup[G]) {
    s.as_set.cardinality_at_most(G.order)
}

/// Every subsemigroup of a finite group is finitely generated.
theorem subsemigroup_of_finite_group_is_finitely_generated[G: FiniteGroup](s: Subsemigroup[G]) {
    subsemigroup_is_finitely_generated(s)
}

theorem submonoid_of_finite_group_is_finite[G: FiniteGroup](s: Submonoid[G]) {
    finite_constraint(s.contains)
}

theorem submonoid_as_set_is_finite[G: FiniteGroup](s: Submonoid[G]) {
    s.as_set.is_finite
}

theorem submonoid_as_set_cardinality_at_most_group_order[G: FiniteGroup](s: Submonoid[G]) {
    s.as_set.cardinality_at_most(G.order)
}

/// Every submonoid of a finite group is finitely generated.
theorem submonoid_of_finite_group_is_finitely_generated[G: FiniteGroup](s: Submonoid[G]) {
    submonoid_is_finitely_generated(s)
}

theorem subgroup_of_finite_group_is_finite[G: FiniteGroup](s: Subgroup[G]) {
    finite_constraint(s.contains)
}

theorem subgroup_as_set_is_finite[G: FiniteGroup](s: Subgroup[G]) {
    s.as_set.is_finite
}

theorem subgroup_as_set_cardinality_at_most_group_order[G: FiniteGroup](s: Subgroup[G]) {
    s.as_set.cardinality_at_most(G.order)
}

/// Every subgroup of a finite group is finitely generated.
theorem subgroup_of_finite_group_is_finitely_generated[G: FiniteGroup](s: Subgroup[G]) {
    subgroup_is_finitely_generated(s)
}

/// A finite subgroup of a finite group.
structure FiniteSubgroup[G: FiniteGroup] {
    /// A list containing all elements of the subgroup.
    elements: List[G]
} constraint {
    elements.is_unique and subgroup_constraint(elements.contains)
}

let finite_subgroup_as_subgroup[G: FiniteGroup](s: FiniteSubgroup[G]) -> result: Subgroup[G] satisfy {
    Subgroup.new(s.elements.contains) = Option.some(result)
}

attributes FiniteSubgroup[G: FiniteGroup] {
    /// The number of elements in the subgroup.
    define order(self) -> Nat {
        self.elements.length
    }

    /// The subgroup determined by the elements of this finite subgroup.
    let as_subgroup: FiniteSubgroup[G] -> Subgroup[G] = finite_subgroup_as_subgroup

    /// Membership predicate.
    define contains(self, x: G) -> Bool {
        self.as_subgroup.contains(x)
    }

    /// The subset of group elements belonging to this finite subgroup.
    define as_set(self) -> Set[G] {
        self.as_subgroup.as_set
    }
}

/// Membership in the associated subgroup is membership in the finite subgroup list.
theorem finite_subgroup_as_subgroup_contains_eq[G: FiniteGroup](s: FiniteSubgroup[G], x: G) {
    s.as_subgroup.contains(x) = s.elements.contains(x)
}

/// Membership in a finite subgroup is membership in its associated subgroup.
theorem finite_subgroup_contains_as_subgroup_eq[G: FiniteGroup](s: FiniteSubgroup[G], x: G) {
    s.contains(x) = s.as_subgroup.contains(x)
}

/// Membership in a finite subgroup is membership in the finite subgroup list.
theorem finite_subgroup_contains_eq[G: FiniteGroup](s: FiniteSubgroup[G], x: G) {
    s.contains(x) = s.elements.contains(x)
}

/// Membership in the finite subgroup list is membership in the associated subgroup.
theorem finite_subgroup_elements_contains_as_subgroup_eq[G: FiniteGroup](s: FiniteSubgroup[G], x: G) {
    s.elements.contains(x) = s.as_subgroup.contains(x)
}

/// Membership in the underlying set is membership in the finite subgroup list.
theorem finite_subgroup_as_set_contains_eq[G: FiniteGroup](s: FiniteSubgroup[G], x: G) {
    s.as_set.contains(x) = s.elements.contains(x)
}

/// Membership in the underlying set is membership in the finite subgroup.
theorem finite_subgroup_as_set_contains_finite_subgroup_eq[G: FiniteGroup](s: FiniteSubgroup[G], x: G) {
    s.as_set.contains(x) = s.contains(x)
}

/// Membership in the finite subgroup list is membership in the underlying set.
theorem finite_subgroup_elements_contains_as_set_eq[G: FiniteGroup](s: FiniteSubgroup[G], x: G) {
    s.elements.contains(x) = s.as_set.contains(x)
}

/// The underlying set of a finite subgroup is the set associated to its element list.
theorem finite_subgroup_as_set_eq_list_set[G: FiniteGroup](s: FiniteSubgroup[G]) {
    s.as_set = list_set(s.elements)
}

/// The set underlying a finite subgroup is finite.
theorem finite_subgroup_as_set_is_finite[G: FiniteGroup](s: FiniteSubgroup[G]) {
    s.as_set.is_finite
}

/// The finite set determined by a finite subgroup.
let finite_subgroup_as_finite_set[G: FiniteGroup](s: FiniteSubgroup[G]) -> result: FiniteSet[G] satisfy {
    FiniteSet.new(s.as_set) = Option.some(result)
}

/// The finite set associated to a finite subgroup has the same underlying set.
theorem finite_subgroup_as_finite_set_as_set[G: FiniteGroup](s: FiniteSubgroup[G]) {
    finite_subgroup_as_finite_set(s).as_set = s.as_set
}

/// Membership in the associated finite set is membership in the finite subgroup list.
theorem finite_subgroup_as_finite_set_contains_eq[G: FiniteGroup](s: FiniteSubgroup[G], x: G) {
    finite_subgroup_as_finite_set(s).contains(x) = s.elements.contains(x)
}

attributes FiniteSubgroup[G: FiniteGroup] {
    /// The finite set of elements of this finite subgroup.
    let as_finite_set: FiniteSubgroup[G] -> FiniteSet[G] = finite_subgroup_as_finite_set
}

theorem identity_list_is_unique[G: FiniteGroup] {
    List.singleton[G](G.1).is_unique
}

theorem identity_meets_identity_constraint[G: FiniteGroup] {
    identity_constraint(List.singleton[G](G.1).contains)
}

theorem identity_meets_closure_constraint[G: FiniteGroup] {
    closure_constraint(List.singleton[G](G.1).contains)
}

theorem identity_meets_inverse_constraint[G: FiniteGroup] {
    inverse_constraint(List.singleton[G](G.1).contains)
}

theorem identity_meets_subgroup_constraint[G: FiniteGroup] {
    subgroup_constraint(List.singleton[G](G.1).contains)
}

theorem identity_meets_fs_constraint[G: FiniteGroup] {
    FiniteSubgroup.constraint(List.singleton[G](G.1))
}

let isg[G: FiniteGroup]: FiniteSubgroup[G] satisfy {
    FiniteSubgroup.new(List.singleton[G](G.1)) = Option.some(isg)
}

theorem subgroup_has_order_at_most_G_order[G: FiniteGroup](s: FiniteSubgroup[G]) {
    s.order <= G.order
}

theorem finite_subgroup_as_set_cardinality_is_order[G: FiniteGroup](s: FiniteSubgroup[G]) {
    s.as_set.cardinality_is(s.order)
}

/// The finite set associated to a finite subgroup has cardinality equal to its order.
theorem finite_subgroup_as_finite_set_cardinality_is_order[G: FiniteGroup](s: FiniteSubgroup[G]) {
    s.as_finite_set.cardinality_is(s.order)
}

/// The order of a finite subgroup is the cardinality of its associated finite set.
theorem finite_subgroup_order_eq_cardinality[G: FiniteGroup](s: FiniteSubgroup[G], n: Nat) {
    s.as_finite_set.cardinality_is(n) implies n = s.order
}

theorem finite_subgroup_as_set_cardinality_at_most_group_order[G: FiniteGroup](s: FiniteSubgroup[G]) {
    s.as_set.cardinality_at_most(G.order)
}

/// The order of a finite subgroup is positive.
theorem finite_subgroup_order_pos[G: FiniteGroup](s: FiniteSubgroup[G]) {
    s.order > Nat.0
}

/// The stabilizer of a point in a finite group action, bundled as a finite subgroup.
let finite_stabilizer[G: FiniteGroup, X](a: MulAction[G, X], x: X) -> result: FiniteSubgroup[G] satisfy {
    FiniteSubgroup.new(G.elements.filter(stabilizer(a, x).contains)) = Option.some(result)
}

/// Membership in the finite stabilizer is the usual stabilizer membership.
theorem finite_stabilizer_contains_eq[G: FiniteGroup, X](
    a: MulAction[G, X],
    x: X,
    g: G
) {
    finite_stabilizer(a, x).contains(g) = stabilizer(a, x).contains(g)
}

/// Membership in the finite stabilizer is exactly fixing the point.
theorem finite_stabilizer_fixes_eq[G: FiniteGroup, X](
    a: MulAction[G, X],
    x: X,
    g: G
) {
    finite_stabilizer(a, x).contains(g) = (a.act(g, x) = x)
}

/// The underlying set of the finite stabilizer has cardinality its order.
theorem finite_stabilizer_as_set_cardinality_is_order[G: FiniteGroup, X](
    a: MulAction[G, X],
    x: X
) {
    finite_stabilizer(a, x).as_set.cardinality_is(finite_stabilizer(a, x).order)
}

/// Two group elements act equally on a point exactly when their quotient lies in the finite stabilizer.
theorem orbit_action_eq_iff_finite_stabilizer_left_coset[G: FiniteGroup, X](
    a: MulAction[G, X],
    x: X,
    g: G,
    k: G
) {
    (a.act(g, x) = a.act(k, x)) = finite_stabilizer(a, x).contains(g.inverse * k)
}

/// The orbit of a point in a finite group action, listed without duplicate values.
define finite_orbit_list[G: FiniteGroup, X](a: MulAction[G, X], x: X) -> List[X] {
    map[G, X](G.elements, function(g: G) { a.act(g, x) }).unique
}

/// The finite orbit list represents exactly the orbit of the point.
theorem finite_orbit_list_set_eq_orbit[G: FiniteGroup, X](a: MulAction[G, X], x: X) {
    list_set(finite_orbit_list(a, x)) = orbit(a, x)
}

/// The orbit of a point in a finite group action has cardinality the finite orbit-list length.
theorem finite_orbit_cardinality_is_length[G: FiniteGroup, X](a: MulAction[G, X], x: X) {
    orbit(a, x).cardinality_is(finite_orbit_list(a, x).length)
}

/// True when a group element belongs to a finite left coset.
define finite_left_coset_contains[G: FiniteGroup](s: FiniteSubgroup[G], a: G, y: G) -> Bool {
    s.contains(a.inverse * y)
}

/// The left coset of a finite subgroup represented as a finite set predicate.
define finite_left_coset_set[G: FiniteGroup](s: FiniteSubgroup[G], a: G) -> Set[G] {
    Set[G].new(finite_left_coset_contains(s, a))
}

/// Membership in a finite left coset is membership of the quotient in the finite subgroup.
theorem finite_left_coset_set_contains_eq[G: FiniteGroup](s: FiniteSubgroup[G], a: G, y: G) {
    finite_left_coset_set(s, a).contains(y) = s.contains(a.inverse * y)
}

/// A representative belongs to its finite left coset.
theorem finite_left_coset_set_contains_representative[G: FiniteGroup](s: FiniteSubgroup[G], a: G) {
    finite_left_coset_set(s, a).contains(a)
}

/// Same-left-coset containment is symmetric.
theorem same_left_coset_contains_symmetric[G: Group](s: Subgroup[G], a: G, b: G) {
    s.contains(a.inverse * b) implies s.contains(b.inverse * a)
}

/// Same-left-coset containment is transitive.
theorem same_left_coset_contains_transitive[G: Group](s: Subgroup[G], a: G, b: G, c: G) {
    s.contains(a.inverse * b) and s.contains(b.inverse * c) implies s.contains(a.inverse * c)
}

/// Representatives in the same left coset have identical coset-membership predicates.
theorem same_left_coset_contains_membership_eq[G: Group](s: Subgroup[G], a: G, b: G, y: G) {
    s.contains(a.inverse * b) implies s.contains(a.inverse * y) = s.contains(b.inverse * y)
}

/// Two finite left-coset predicates are equal exactly when the representative quotient lies in the subgroup.
theorem finite_left_coset_set_eq_iff_contains_quotient[G: FiniteGroup](s: FiniteSubgroup[G], a: G, b: G) {
    (finite_left_coset_set(s, a) = finite_left_coset_set(s, b)) = s.contains(a.inverse * b)
}

/// The duplicate-free list of finite left cosets of a finite subgroup.
define finite_left_coset_representatives[G: FiniteGroup](s: FiniteSubgroup[G]) -> List[Set[G]] {
    map[G, Set[G]](G.elements, function(a: G) { finite_left_coset_set(s, a) }).unique
}

/// Every group element contributes its finite left coset to the representative list.
theorem finite_left_coset_representatives_contains_coset[G: FiniteGroup](s: FiniteSubgroup[G], a: G) {
    finite_left_coset_representatives(s).contains(finite_left_coset_set(s, a))
}

/// The orbit map and finite-stabilizer left-coset map have the same equality kernel.
theorem finite_orbit_action_eq_iff_finite_stabilizer_left_coset_set_eq[G: FiniteGroup, X](
    a: MulAction[G, X],
    x: X,
    g: G,
    k: G
) {
    (a.act(g, x) = a.act(k, x)) =
        (finite_left_coset_set(finite_stabilizer(a, x), g) = finite_left_coset_set(finite_stabilizer(a, x), k))
}

/// A finite left coset has the cardinality of its finite subgroup.
theorem finite_left_coset_set_cardinality_is_order[G: FiniteGroup](s: FiniteSubgroup[G], a: G) {
    finite_left_coset_set(s, a).cardinality_is(s.order)
}

/// Any two finite left cosets of a finite subgroup are equal or disjoint.
theorem finite_left_cosets_equal_or_disjoint[G: FiniteGroup](s: FiniteSubgroup[G], a: G, b: G) {
    finite_left_coset_set(s, a) = finite_left_coset_set(s, b) or
    finite_left_coset_set(s, a).is_disjoint(finite_left_coset_set(s, b))
}

/// The finite left-coset representative list is duplicate-free.
theorem finite_left_coset_representatives_is_unique[G: FiniteGroup](s: FiniteSubgroup[G]) {
    finite_left_coset_representatives(s).is_unique
}

/// Every listed finite left coset has cardinality equal to the subgroup order.
theorem finite_left_coset_representative_cardinality_is_order[G: FiniteGroup](s: FiniteSubgroup[G], c: Set[G]) {
    finite_left_coset_representatives(s).contains(c) implies c.cardinality_is(s.order)
}

/// The recursive union of finite left-coset representatives is the universal set.
theorem finite_left_coset_representatives_union_covers_universal[G: FiniteGroup](s: FiniteSubgroup[G]) {
    set_list_union[G](finite_left_coset_representatives(s)) = Set[G].universal_set
}

/// Finite left-coset representatives exactly cover the whole finite group.
theorem finite_left_coset_representatives_exactly_cover_universal[G: FiniteGroup](
    s: FiniteSubgroup[G]
) {
    set_list_exactly_covers[G](finite_left_coset_representatives(s), Set[G].universal_set)
}

/// Finite left-coset representatives form a partition of the whole finite group.
theorem finite_left_coset_representatives_partition_of_universal[G: FiniteGroup](
    s: FiniteSubgroup[G]
) {
    set_list_partition_of[G](finite_left_coset_representatives(s), Set[G].universal_set)
}

/// The finite left-coset partition is uniform, with each part having subgroup order.
theorem finite_left_coset_partition_uniform_cardinality[G: FiniteGroup](s: FiniteSubgroup[G]) {
    set_list_partition_of[G](finite_left_coset_representatives(s), Set[G].universal_set) and
    set_list_all_cardinality_is[G](finite_left_coset_representatives(s), s.order)
}

/// The subgroup order times the number of finite left cosets equals the group order.
theorem finite_left_coset_representatives_order_product_eq_group_order[G: FiniteGroup](s: FiniteSubgroup[G]) {
    s.order * finite_left_coset_representatives(s).length = G.order
}

/// Lagrange's theorem: the order of a finite subgroup divides the group order.
theorem finite_subgroup_order_divides_group_order[G: FiniteGroup](s: FiniteSubgroup[G]) {
    s.order.divides(G.order)
}

/// The finite orbit list has the same length as the finite-stabilizer left-coset representative list.
theorem finite_orbit_list_length_eq_finite_stabilizer_cosets[G: FiniteGroup, X](
    a: MulAction[G, X],
    x: X
) {
    finite_orbit_list[G, X](a, x).length =
        finite_left_coset_representatives[G](finite_stabilizer[G, X](a, x)).length
}

/// Orbit-stabilizer product formula for the finite orbit list.
theorem finite_stabilizer_order_mul_finite_orbit_list_length_eq_group_order[G: FiniteGroup, X](
    a: MulAction[G, X],
    x: X
) {
    finite_stabilizer[G, X](a, x).order * finite_orbit_list[G, X](a, x).length = G.order
}

theorem all_elements_have_order_at_most_G[G: FiniteGroup](g: G) {
    exists (n: Nat) {
        n > 0 and n <= G.order and g.pow(n) = G.1
    }
}

theorem all_elements_have_finite_order[G: FiniteGroup](g: G) {
    has_finite_order(g)
}

theorem pow_mod_order[G: Group](g: G, m: Nat, k: Nat) {
    m > 0 and g.pow(m) = G.1 implies g.pow(k.mod(m)) = g.pow(k)
}

theorem inverse_is_power[G: Group](g: G, m: Nat) {
    m > 0 and g.pow(m) = G.1 implies g.inverse = g.pow(m - 1)
}

theorem cyclic_powers_contains[G: FiniteGroup](g: G, k: Nat) {
    map(G.order.range, g.pow).unique.contains(g.pow(k))
}

theorem cyclic_subgroup_constraint[G: FiniteGroup](g: G) {
    FiniteSubgroup.constraint(map(G.order.range, g.pow).unique)
}

let cyclic_subgroup_of[G: FiniteGroup](g: G) -> result: FiniteSubgroup[G] satisfy {
    FiniteSubgroup.new(map(G.order.range, g.pow).unique) = Option.some(result)
}

/// The elements of the cyclic finite subgroup are the unique powers in the bounded power list.
theorem cyclic_subgroup_elements_eq[G: FiniteGroup](g: G) {
    cyclic_subgroup_of(g).elements = map(G.order.range, g.pow).unique
}

/// The order of the finite cyclic subgroup generated by an element divides the group order.
theorem finite_cyclic_subgroup_order_divides_group_order[G: FiniteGroup](g: G) {
    cyclic_subgroup_of(g).order.divides(G.order)
}

/// The order of the cyclic finite subgroup generated by an element is positive.
theorem cyclic_subgroup_order_pos[G: FiniteGroup](g: G) {
    cyclic_subgroup_of(g).order > Nat.0
}

/// Every power of an element belongs to its cyclic finite subgroup.
theorem cyclic_subgroup_contains_power[G: FiniteGroup](g: G, k: Nat) {
    cyclic_subgroup_of(g).contains(g.pow(k))
}

/// The order of the cyclic finite subgroup is at most any finite subgroup containing the generator.
theorem cyclic_subgroup_order_le_of_contains_generator[G: FiniteGroup](g: G, s: FiniteSubgroup[G]) {
    s.contains(g) implies cyclic_subgroup_of(g).order <= s.order
}

/// A positive exponent that sends the generator to identity bounds the cyclic subgroup order.
theorem cyclic_subgroup_order_le_period[G: FiniteGroup](g: G, m: Nat) {
    m > Nat.0 and g.pow(m) = G.1 implies cyclic_subgroup_of(g).order <= m
}

/// No positive exponent below the cyclic subgroup order sends the generator to identity.
theorem no_positive_period_below_cyclic_subgroup_order[G: FiniteGroup](g: G, k: Nat) {
    k > Nat.0 and k < cyclic_subgroup_of(g).order implies g.pow(k) != G.1
}

/// The cyclic finite subgroup order exponent sends its generator to the identity.
theorem pow_cyclic_subgroup_order_eq_identity[G: FiniteGroup](g: G) {
    g.pow(cyclic_subgroup_of(g).order) = G.1
}

/// Powers of an element repeat modulo the order of its cyclic finite subgroup.
theorem pow_mod_cyclic_subgroup_order[G: FiniteGroup](g: G, k: Nat) {
    g.pow(k.mod(cyclic_subgroup_of(g).order)) = g.pow(k)
}

/// The inverse of an element is the preceding power in its cyclic finite period.
theorem inverse_eq_power_cyclic_subgroup_order_pred[G: FiniteGroup](g: G) {
    g.inverse = g.pow(cyclic_subgroup_of(g).order - Nat.1)
}

/// Adding the cyclic finite subgroup order to an exponent leaves the power unchanged.
theorem pow_add_cyclic_subgroup_order[G: FiniteGroup](g: G, k: Nat) {
    g.pow(k + cyclic_subgroup_of(g).order) = g.pow(k)
}

/// Every power of an element belongs to the underlying set of its cyclic finite subgroup.
theorem cyclic_subgroup_as_set_contains_power[G: FiniteGroup](g: G, k: Nat) {
    cyclic_subgroup_of(g).as_set.contains(g.pow(k))
}

/// Every power of an element belongs to the finite set of its cyclic finite subgroup.
theorem cyclic_subgroup_as_finite_set_contains_power[G: FiniteGroup](g: G, k: Nat) {
    cyclic_subgroup_of(g).as_finite_set.contains(g.pow(k))
}

/// An element belongs to its cyclic finite subgroup.
theorem cyclic_subgroup_contains_generator[G: FiniteGroup](g: G) {
    cyclic_subgroup_of(g).contains(g)
}

/// The underlying set of the cyclic finite subgroup contains its generator.
theorem cyclic_subgroup_as_set_contains_generator[G: FiniteGroup](g: G) {
    cyclic_subgroup_of(g).as_set.contains(g)
}

/// The finite set of the cyclic finite subgroup contains its generator.
theorem cyclic_subgroup_as_finite_set_contains_generator[G: FiniteGroup](g: G) {
    cyclic_subgroup_of(g).as_finite_set.contains(g)
}

/// The cyclic subgroup order is a positive period, and powers repeat after adding it.
theorem cyclic_order_periodic_power[G: FiniteGroup](g: G, k: Nat) {
    cyclic_subgroup_of(g).order > Nat.0 and
    g.pow(cyclic_subgroup_of(g).order) = G.1 and
    g.pow(k + cyclic_subgroup_of(g).order) = g.pow(k)
}

/// Reducing an exponent modulo the cyclic subgroup order preserves the power and stays in the cyclic subgroup.
theorem cyclic_order_mod_period_power[G: FiniteGroup](g: G, k: Nat) {
    cyclic_subgroup_of(g).order > Nat.0 and
    g.pow(k.mod(cyclic_subgroup_of(g).order)) = g.pow(k) and
    cyclic_subgroup_of(g).contains(g.pow(k.mod(cyclic_subgroup_of(g).order))) and
    cyclic_subgroup_of(g).contains(g.pow(k))
}

/// The generator occurs as the first positive power and belongs to each bundled cyclic carrier.
theorem cyclic_order_generator_power_one[G: FiniteGroup](g: G) {
    g.pow(Nat.1) = g and
    cyclic_subgroup_of(g).contains(g) and
    cyclic_subgroup_of(g).as_set.contains(g) and
    cyclic_subgroup_of(g).as_finite_set.contains(g)
}

/// The cyclic subgroup order is a positive identity power, and that identity lies in the generated subgroup.
theorem cyclic_order_identity_power[G: FiniteGroup](g: G) {
    cyclic_subgroup_of(g).order > Nat.0 and
    g.pow(cyclic_subgroup_of(g).order) = G.1 and
    cyclic_subgroup_of(g).contains(G.1)
}

/// A positive identity power cannot be smaller than the cyclic subgroup order.
theorem cyclic_order_no_smaller_positive_period[G: FiniteGroup](g: G, k: Nat) {
    k > Nat.0 and g.pow(k) = G.1 implies
        cyclic_subgroup_of(g).order <= k and not k < cyclic_subgroup_of(g).order
}

/// The subgroup associated to a cyclic finite subgroup is finitely generated.
theorem cyclic_subgroup_as_subgroup_is_finitely_generated[G: FiniteGroup](g: G) {
    subgroup_is_finitely_generated(cyclic_subgroup_of(g).as_subgroup)
}

attributes G: FiniteGroup {
    /// The trivial subgroup containing only the identity element.
    let identity_subgroup: FiniteSubgroup[G] = isg[G]

    /// The cyclic subgroup generated by this element.
    // TODO: prove this actually is the subgroup you'd expect, not a degenerate case.
    define cyclic_subgroup(self) -> FiniteSubgroup[G] {
        cyclic_subgroup_of(self)
    }
}
