from list import List, map
from list import contains_imp_unique_contains, unique_contains_imp_contains, unique_is_smallest_containing_list
from list import map_contains, map_contains_of_contains, map_length, range_contains_of_lt, length_range,
    range_pigeonhole
from algebra.group import Group, has_finite_order, pow_cancel
from nat import Nat, lt_suc, lte_and_lt, lt_diff, add_imp_sub, lte_antisymm,
    lt_suc_right, not_lt_zero, trichotomy
from number_theory import mod_lt
numerals Nat

/// A group is finite if its elements can be placed in a (finite) list
typeclass G: FiniteGroup extends Group {
    /// A list containing all elements of the group.
    elements: List[G]

    /// Every group element appears in the elements list.
    all_group_elements_in_elements(g: G) {
        G.elements.contains_every
    }
    /// The elements list contains no duplicates.
    unique_elements_list {
        G.elements.is_unique
    }
}

attributes G: FiniteGroup {
    /// The number of elements in the group.
    let order: Nat = G.elements.length
}

from algebra.subgroup import Subgroup, subgroup_constraint, identity_constraint, inverse_constraint, closure_constraint,
    subgroup_closure_as_set, subgroup_is_finitely_generated, subgroup_contains_identity
from algebra.subsemigroup import Subsemigroup, subsemigroup_closure_as_set, subsemigroup_is_finitely_generated
from algebra.monoid.submonoid import Submonoid, submonoid_closure_as_set, submonoid_is_finitely_generated
from finite_set import FiniteSet
from data.basic.set import Set, finite_constraint, list_set, list_set_contains_eq,
    unique_list_set_cardinality_is_length, set_ext, cardinality_is_well_defined

theorem subsemigroup_of_finite_group_is_finite[G: FiniteGroup](s: Subsemigroup[G]) {
    finite_constraint(s.contains)
} by {
    forall(x: G) {
        s.contains(x) implies G.elements.contains(x)
    }
}

theorem subsemigroup_as_set_is_finite[G: FiniteGroup](s: Subsemigroup[G]) {
    s.as_set.is_finite
} by {
    subsemigroup_of_finite_group_is_finite(s)
    forall(x: G) {
        s.as_set.contains(x) = s.contains(x)
    }
}

theorem subsemigroup_as_set_cardinality_at_most_group_order[G: FiniteGroup](s: Subsemigroup[G]) {
    s.as_set.cardinality_at_most(G.order)
} by {
    forall(x: G) {
        if s.as_set.contains(x) {
            G.elements.contains(x)
        }
    }
    G.elements.contains_set(s.as_set)
    G.elements.length <= G.order
}

/// Every subsemigroup of a finite group is finitely generated.
theorem subsemigroup_of_finite_group_is_finitely_generated[G: FiniteGroup](s: Subsemigroup[G]) {
    subsemigroup_is_finitely_generated(s)
} by {
    subsemigroup_as_set_is_finite(s)
    subsemigroup_closure_as_set(s)
}

theorem submonoid_of_finite_group_is_finite[G: FiniteGroup](s: Submonoid[G]) {
    finite_constraint(s.contains)
} by {
    forall(x: G) {
        s.contains(x) implies G.elements.contains(x)
    }
}

theorem submonoid_as_set_is_finite[G: FiniteGroup](s: Submonoid[G]) {
    s.as_set.is_finite
} by {
    submonoid_of_finite_group_is_finite(s)
    forall(x: G) {
        s.as_set.contains(x) = s.contains(x)
    }
}

theorem submonoid_as_set_cardinality_at_most_group_order[G: FiniteGroup](s: Submonoid[G]) {
    s.as_set.cardinality_at_most(G.order)
} by {
    forall(x: G) {
        if s.as_set.contains(x) {
            G.elements.contains(x)
        }
    }
}

/// Every submonoid of a finite group is finitely generated.
theorem submonoid_of_finite_group_is_finitely_generated[G: FiniteGroup](s: Submonoid[G]) {
    submonoid_is_finitely_generated(s)
} by {
    submonoid_as_set_is_finite(s)
    submonoid_closure_as_set(s)
}

theorem subgroup_of_finite_group_is_finite[G: FiniteGroup](s: Subgroup[G]) {
    finite_constraint(s.contains)
} by {
    forall(x: G) {
        s.contains(x) implies G.elements.contains(x)
    }
}

theorem subgroup_as_set_is_finite[G: FiniteGroup](s: Subgroup[G]) {
    s.as_set.is_finite
} by {
    subgroup_of_finite_group_is_finite(s)
    forall(x: G) {
        s.as_set.contains(x) = s.contains(x)
    }
}

theorem subgroup_as_set_cardinality_at_most_group_order[G: FiniteGroup](s: Subgroup[G]) {
    s.as_set.cardinality_at_most(G.order)
} by {
    forall(x: G) {
        if s.as_set.contains(x) {
            G.elements.contains(x)
        }
    }
}

/// Every subgroup of a finite group is finitely generated.
theorem subgroup_of_finite_group_is_finitely_generated[G: FiniteGroup](s: Subgroup[G]) {
    subgroup_is_finitely_generated(s)
} by {
    subgroup_as_set_is_finite(s)
    subgroup_closure_as_set(s)
}

/// A finite subgroup of a finite group.
structure FiniteSubgroup[G: FiniteGroup] {
    /// A list containing all elements of the subgroup.
    elements: List[G]
} constraint {
    elements.is_unique and subgroup_constraint(elements.contains)
} by {
    let s = List.singleton[G](G.1)

    forall(x: G) {
        if s.contains(x) {
        } else {
            not identity_subgroup[G].contains(x)
        }
        not List.nil[G].contains(x)
        List.cons(G.1, List.nil[G]) = List.singleton(G.1)
        s.contains(x) = (x = G.1)
        s.contains(x) = identity_subgroup[G].contains(x)
    }
    s.contains = identity_subgroup[G].contains
    subgroup_constraint(s.contains)
}

let finite_subgroup_as_subgroup[G: FiniteGroup](s: FiniteSubgroup[G]) -> result: Subgroup[G] satisfy {
    Subgroup.new(s.elements.contains) = Option.some(result)
}

attributes FiniteSubgroup[G: FiniteGroup] {
    /// The number of elements in the subgroup.
    define order(self) -> Nat {
        self.elements.length
    }

    /// The subgroup determined by the elements of this finite subgroup.
    let as_subgroup: FiniteSubgroup[G] -> Subgroup[G] = finite_subgroup_as_subgroup

    /// Membership predicate.
    define contains(self, x: G) -> Bool {
        self.as_subgroup.contains(x)
    }

    /// The subset of group elements belonging to this finite subgroup.
    define as_set(self) -> Set[G] {
        self.as_subgroup.as_set
    }
}

/// Membership in the associated subgroup is membership in the finite subgroup list.
theorem finite_subgroup_as_subgroup_contains_eq[G: FiniteGroup](s: FiniteSubgroup[G], x: G) {
    s.as_subgroup.contains(x) = s.elements.contains(x)
}

/// Membership in a finite subgroup is membership in its associated subgroup.
theorem finite_subgroup_contains_as_subgroup_eq[G: FiniteGroup](s: FiniteSubgroup[G], x: G) {
    s.contains(x) = s.as_subgroup.contains(x)
}

/// Membership in a finite subgroup is membership in the finite subgroup list.
theorem finite_subgroup_contains_eq[G: FiniteGroup](s: FiniteSubgroup[G], x: G) {
    s.contains(x) = s.elements.contains(x)
} by {
    finite_subgroup_contains_as_subgroup_eq(s, x)
    finite_subgroup_as_subgroup_contains_eq(s, x)
}

/// Membership in the finite subgroup list is membership in the associated subgroup.
theorem finite_subgroup_elements_contains_as_subgroup_eq[G: FiniteGroup](s: FiniteSubgroup[G], x: G) {
    s.elements.contains(x) = s.as_subgroup.contains(x)
} by {
    finite_subgroup_as_subgroup_contains_eq(s, x)
}

/// Membership in the underlying set is membership in the finite subgroup list.
theorem finite_subgroup_as_set_contains_eq[G: FiniteGroup](s: FiniteSubgroup[G], x: G) {
    s.as_set.contains(x) = s.elements.contains(x)
} by {
    finite_subgroup_as_subgroup_contains_eq(s, x)
}

/// Membership in the underlying set is membership in the finite subgroup.
theorem finite_subgroup_as_set_contains_finite_subgroup_eq[G: FiniteGroup](s: FiniteSubgroup[G], x: G) {
    s.as_set.contains(x) = s.contains(x)
} by {
    finite_subgroup_as_set_contains_eq(s, x)
    finite_subgroup_contains_eq(s, x)
}

/// Membership in the finite subgroup list is membership in the underlying set.
theorem finite_subgroup_elements_contains_as_set_eq[G: FiniteGroup](s: FiniteSubgroup[G], x: G) {
    s.elements.contains(x) = s.as_set.contains(x)
} by {
    finite_subgroup_as_set_contains_eq(s, x)
}

/// The underlying set of a finite subgroup is the set associated to its element list.
theorem finite_subgroup_as_set_eq_list_set[G: FiniteGroup](s: FiniteSubgroup[G]) {
    s.as_set = list_set(s.elements)
} by {
    forall(x: G) {
        finite_subgroup_as_set_contains_eq(s, x)
        list_set_contains_eq(s.elements, x)
        s.as_set.contains(x) = list_set(s.elements).contains(x)
    }
    set_ext(s.as_set, list_set(s.elements))
}

/// The set underlying a finite subgroup is finite.
theorem finite_subgroup_as_set_is_finite[G: FiniteGroup](s: FiniteSubgroup[G]) {
    s.as_set.is_finite
} by {
    subgroup_as_set_is_finite(s.as_subgroup)
}

/// The finite set determined by a finite subgroup.
let finite_subgroup_as_finite_set[G: FiniteGroup](s: FiniteSubgroup[G]) -> result: FiniteSet[G] satisfy {
    FiniteSet.new(s.as_set) = Option.some(result)
} by {
    finite_subgroup_as_set_is_finite(s)
}

/// The finite set associated to a finite subgroup has the same underlying set.
theorem finite_subgroup_as_finite_set_as_set[G: FiniteGroup](s: FiniteSubgroup[G]) {
    finite_subgroup_as_finite_set(s).as_set = s.as_set
}

/// Membership in the associated finite set is membership in the finite subgroup list.
theorem finite_subgroup_as_finite_set_contains_eq[G: FiniteGroup](s: FiniteSubgroup[G], x: G) {
    finite_subgroup_as_finite_set(s).contains(x) = s.elements.contains(x)
} by {
    finite_subgroup_as_finite_set(s).contains(x) = finite_subgroup_as_finite_set(s).as_set.contains(x)
    finite_subgroup_as_finite_set_as_set(s)
    finite_subgroup_as_set_contains_eq(s, x)
}

attributes FiniteSubgroup[G: FiniteGroup] {
    /// The finite set of elements of this finite subgroup.
    let as_finite_set: FiniteSubgroup[G] -> FiniteSet[G] = finite_subgroup_as_finite_set
}

theorem identity_list_is_unique[G: FiniteGroup] {
    List.singleton[G](G.1).is_unique
}

theorem identity_meets_identity_constraint[G: FiniteGroup] {
    identity_constraint(List.singleton[G](G.1).contains)
}

theorem identity_meets_closure_constraint[G: FiniteGroup] {
    closure_constraint(List.singleton[G](G.1).contains)
} by {
    let s = List.singleton[G](G.1)
    forall(a: G, b: G) {
        if s.contains(a) and s.contains(b) {
            not List.nil[G].contains(a)
            List.cons(G.1, List.nil[G]) = s
            a = G.1
            b = G.1
            s.contains(a * b)
        }
    }
}

theorem identity_meets_inverse_constraint[G: FiniteGroup] {
    inverse_constraint(List.singleton[G](G.1).contains)
} by {
    let s = List.singleton[G](G.1)
    forall(a: G) {
        if s.contains(a) {
            not List.nil[G].contains(a)
            List.cons(G.1, List.nil[G]) = s
            a = G.1
            G.1.inverse = G.1
            s.contains(a.inverse)
        }
    }
}

theorem identity_meets_subgroup_constraint[G: FiniteGroup] {
    subgroup_constraint(List.singleton[G](G.1).contains)
}

theorem identity_meets_fs_constraint[G: FiniteGroup] {
    FiniteSubgroup.constraint(List.singleton[G](G.1))
}

let isg[G: FiniteGroup]: FiniteSubgroup[G] satisfy {
    FiniteSubgroup.new(List.singleton[G](G.1)) = Option.some(isg)
}

theorem subgroup_has_order_at_most_G_order[G: FiniteGroup](s: FiniteSubgroup[G]) {
    s.order <= G.order
} by {
    s.elements.unique.length <= G.elements.length
    s.elements.is_unique
}

theorem finite_subgroup_as_set_cardinality_is_order[G: FiniteGroup](s: FiniteSubgroup[G]) {
    s.as_set.cardinality_is(s.order)
} by {
    unique_list_set_cardinality_is_length(s.elements)
    finite_subgroup_as_set_eq_list_set(s)
    s.as_set.cardinality_is(s.elements.length)
}

/// The finite set associated to a finite subgroup has cardinality equal to its order.
theorem finite_subgroup_as_finite_set_cardinality_is_order[G: FiniteGroup](s: FiniteSubgroup[G]) {
    s.as_finite_set.cardinality_is(s.order)
} by {
    finite_subgroup_as_set_cardinality_is_order(s)
    s.as_set.cardinality_is(s.order)
    s.as_finite_set.as_set = s.as_set
    s.as_finite_set.as_set.cardinality_is(s.order)
    s.as_finite_set.cardinality_is(s.order)
}

/// The order of a finite subgroup is the cardinality of its associated finite set.
theorem finite_subgroup_order_eq_cardinality[G: FiniteGroup](s: FiniteSubgroup[G], n: Nat) {
    s.as_finite_set.cardinality_is(n) implies n = s.order
} by {
    if s.as_finite_set.cardinality_is(n) {
        finite_subgroup_as_finite_set_cardinality_is_order(s)
        s.as_finite_set.underlying_set.cardinality_is(n)
        s.as_finite_set.underlying_set.cardinality_is(s.order)
        cardinality_is_well_defined(s.as_finite_set.underlying_set, n, s.order)
        n = s.order
    }
}

theorem finite_subgroup_as_set_cardinality_at_most_group_order[G: FiniteGroup](s: FiniteSubgroup[G]) {
    s.as_set.cardinality_at_most(G.order)
} by {
    subgroup_as_set_cardinality_at_most_group_order(s.as_subgroup)
}

/// The order of a finite subgroup is positive.
theorem finite_subgroup_order_pos[G: FiniteGroup](s: FiniteSubgroup[G]) {
    s.order > Nat.0
} by {
    subgroup_contains_identity(s.as_subgroup)
    finite_subgroup_as_subgroup_contains_eq(s, G.1)
    match s.elements {
        List.nil {
        }
        List.cons(head, tail) {
            s.order = tail.length.suc
            lt_suc(Nat.0)
            Nat.0 < tail.length.suc
            s.order > Nat.0
        }
    }
}

theorem all_elements_have_order_at_most_G[G: FiniteGroup](g: G) {
    exists (n: Nat) {
        n > 0 and n <= G.order and g.pow(n) = G.1
    }
} by {
    // Proof idea: apply pigeonhole. G has |G| elements, take n = |G|+1 powers
    // of g, [1, g, ..., g.pow(n)], which means two elements at some indices i <
    // j match. Then `g.pow(i) = g.pow(j)` so we must have `g.pow(j - i) = G.1`,
    // or that the order is <= j - i <= |G|.
    let n = G.order + 1
    let f = g.pow
    let cyclic_subgroup = map(n.range, f)

    map[Nat, G](n.range, f).length = n.range.length
    n.range.length = n
    cyclic_subgroup.length = n

    // Prover help
    forall(x: G) {
        cyclic_subgroup.contains(x) implies G.elements.contains(x)
    }
    cyclic_subgroup.unique.length <= G.elements.length
    map(n.range, f).unique.length <= G.elements.length
    cyclic_subgroup.unique.length <= G.elements.length
    G.elements.length < n
    cyclic_subgroup.unique.length < n
    cyclic_subgroup.unique.length < cyclic_subgroup.length
    map(n.range, f).unique.length < n

    let (i: Nat, j: Nat) satisfy {
        i < j and j < n and f(i) = f(j)
    }

    j < cyclic_subgroup.length

    // Weird prover help stuff
    let m = j - i
    i + m = j
    // Since m + i = j < n
    m < n

    // Conclusion
    m <= G.order
    g.pow(m) = G.1
}

theorem all_elements_have_finite_order[G: FiniteGroup](g: G) {
    has_finite_order(g)
} by {
    let (n: Nat) satisfy {
        n > 0 and n <= G.order and g.pow(n) = G.1
    }
}

theorem pow_mod_order[G: Group](g: G, m: Nat, k: Nat) {
    m > 0 and g.pow(m) = G.1 implies g.pow(k.mod(m)) = g.pow(k)
} by {
    if m > 0 and g.pow(m) = G.1 {
        let q: Nat satisfy {
            q * m + k.mod(m) = k
        }
        g.pow(m).pow(q) = G.1.pow(q)
        g.pow(q * m) = G.1
        g.pow(q * m) * g.pow(k.mod(m)) = g.pow(q * m + k.mod(m))
        g.pow(k.mod(m)) = g.pow(k)
    }
}

theorem inverse_is_power[G: Group](g: G, m: Nat) {
    m > 0 and g.pow(m) = G.1 implies g.inverse = g.pow(m - 1)
} by {
    if m > 0 and g.pow(m) = G.1 {
        1 <= m
        m - 1 + 1 = m
        g.pow(m - 1) * g.pow(1) = g.pow(m - 1 + 1)
        g.pow(1) = g
        g.pow(m - 1) * g = g.pow(m)
        g.pow(m - 1) * g = G.1
        g.inverse * g = G.1
        g.pow(m - 1) * g = g.inverse * g
        g.pow(m - 1) = g.inverse
        g.inverse = g.pow(m - 1)
    }
}

theorem cyclic_powers_contains[G: FiniteGroup](g: G, k: Nat) {
    map(G.order.range, g.pow).unique.contains(g.pow(k))
} by {
    let (m: Nat) satisfy {
        m > 0 and m <= G.order and g.pow(m) = G.1
    }
    m != Nat.0
    k.mod(m) < m
    k.mod(m) < G.order
    map_contains_of_contains(G.order.range, g.pow, k.mod(m))
    map(G.order.range, g.pow).contains(g.pow(k.mod(m)))
    pow_mod_order(g, m, k)
    g.pow(k.mod(m)) = g.pow(k)
    contains_imp_unique_contains(map(G.order.range, g.pow), g.pow(k))
}

theorem cyclic_subgroup_constraint[G: FiniteGroup](g: G) {
    FiniteSubgroup.constraint(map(G.order.range, g.pow).unique)
} by {
    let powers = map(G.order.range, g.pow)
    let unique_powers = powers.unique
    let (m: Nat) satisfy {
        m > 0 and m <= G.order and g.pow(m) = G.1
    }

    unique_powers.is_unique

    cyclic_powers_contains(g, Nat.0)
    unique_powers.contains(G.1)

    forall(a: G, b: G) {
        if unique_powers.contains(a) and unique_powers.contains(b) {
            unique_contains_imp_contains(powers, a)
            unique_contains_imp_contains(powers, b)
            map_contains(G.order.range, g.pow, a)
            let i: Nat satisfy {
                G.order.range.contains(i) and g.pow(i) = a
            }
            map_contains(G.order.range, g.pow, b)
            let j: Nat satisfy {
                G.order.range.contains(j) and g.pow(j) = b
            }
            g.pow(i) * g.pow(j) = g.pow(i + j)
            cyclic_powers_contains(g, i + j)
            unique_powers.contains(a * b)
        }
    }
    closure_constraint(unique_powers.contains)

    inverse_is_power(g, m)
    forall(a: G) {
        if unique_powers.contains(a) {
            unique_contains_imp_contains(powers, a)
            map_contains(G.order.range, g.pow, a)
            let i: Nat satisfy {
                G.order.range.contains(i) and g.pow(i) = a
            }
            g.inverse.pow(i) = g.pow(m - 1).pow(i)
            g.pow(m - 1).pow(i) = g.pow((m - 1) * i)
            cyclic_powers_contains(g, (m - 1) * i)
            unique_powers.contains(a.inverse)
        }
    }
    inverse_constraint(unique_powers.contains)

}

let cyclic_subgroup_of[G: FiniteGroup](g: G) -> result: FiniteSubgroup[G] satisfy {
    FiniteSubgroup.new(map(G.order.range, g.pow).unique) = Option.some(result)
} by {
    cyclic_subgroup_constraint(g)
}

/// The elements of the cyclic finite subgroup are the unique powers in the bounded power list.
theorem cyclic_subgroup_elements_eq[G: FiniteGroup](g: G) {
    cyclic_subgroup_of(g).elements = map(G.order.range, g.pow).unique
}

/// The order of the cyclic finite subgroup generated by an element is positive.
theorem cyclic_subgroup_order_pos[G: FiniteGroup](g: G) {
    cyclic_subgroup_of(g).order > Nat.0
} by {
    finite_subgroup_order_pos(cyclic_subgroup_of(g))
}

/// Every power of an element belongs to its cyclic finite subgroup.
theorem cyclic_subgroup_contains_power[G: FiniteGroup](g: G, k: Nat) {
    cyclic_subgroup_of(g).contains(g.pow(k))
} by {
    cyclic_subgroup_elements_eq(g)
    cyclic_powers_contains(g, k)
    finite_subgroup_as_subgroup_contains_eq(cyclic_subgroup_of(g), g.pow(k))
}

/// The order of the cyclic finite subgroup is at most any finite subgroup containing the generator.
theorem cyclic_subgroup_order_le_of_contains_generator[G: FiniteGroup](g: G, s: FiniteSubgroup[G]) {
    s.contains(g) implies cyclic_subgroup_of(g).order <= s.order
} by {
    if s.contains(g) {
        let powers = map(G.order.range, g.pow)
        let unique_powers = powers.unique
        cyclic_subgroup_elements_eq(g)
        finite_subgroup_contains_as_subgroup_eq(s, g)
        forall(x: G) {
            if powers.contains(x) {
                map_contains(G.order.range, g.pow, x)
                let k: Nat satisfy {
                    G.order.range.contains(k) and g.pow(k) = x
                }
                finite_subgroup_as_subgroup_contains_eq(s, g.pow(k))
                s.elements.contains(g.pow(k))
                s.elements.contains(x)
            }
        }
        unique_is_smallest_containing_list(powers, s.elements)
        powers.unique.length <= s.elements.length
        cyclic_subgroup_of(g).order <= s.order
    }
}

/// A positive exponent that sends the generator to identity bounds the cyclic subgroup order.
theorem cyclic_subgroup_order_le_period[G: FiniteGroup](g: G, m: Nat) {
    m > Nat.0 and g.pow(m) = G.1 implies cyclic_subgroup_of(g).order <= m
} by {
    if m > Nat.0 and g.pow(m) = G.1 {
        m != Nat.0
        let powers = map(G.order.range, g.pow)
        let period_powers = map(m.range, g.pow)
        cyclic_subgroup_elements_eq(g)
        forall(x: G) {
            if powers.contains(x) {
                map_contains(G.order.range, g.pow, x)
                let k: Nat satisfy {
                    G.order.range.contains(k) and g.pow(k) = x
                }
                mod_lt(k, m)
                range_contains_of_lt(m, k.mod(m))
                map_contains_of_contains(m.range, g.pow, k.mod(m))
                period_powers.contains(g.pow(k.mod(m)))
                pow_mod_order(g, m, k)
                g.pow(k.mod(m)) = g.pow(k)
                period_powers.contains(x)
            }
        }
        unique_is_smallest_containing_list(powers, period_powers)
        powers.unique.length <= period_powers.length
        map_length(m.range, g.pow)
        length_range(m)
        cyclic_subgroup_of(g).order <= m
    }
}

/// No positive exponent below the cyclic subgroup order sends the generator to identity.
theorem no_positive_period_below_cyclic_subgroup_order[G: FiniteGroup](g: G, k: Nat) {
    k > Nat.0 and k < cyclic_subgroup_of(g).order implies g.pow(k) != G.1
} by {
    if k > Nat.0 and k < cyclic_subgroup_of(g).order {
        if g.pow(k) = G.1 {
            cyclic_subgroup_order_le_period(g, k)
            false
        }
    }
}

/// The cyclic finite subgroup order exponent sends its generator to the identity.
theorem pow_cyclic_subgroup_order_eq_identity[G: FiniteGroup](g: G) {
    g.pow(cyclic_subgroup_of(g).order) = G.1
} by {
    let s: FiniteSubgroup[G] = cyclic_subgroup_of(g)
    let n: Nat = s.order + Nat.1
    let powers = map(n.range, g.pow)
    cyclic_subgroup_order_pos(g)
    forall(x: G) {
        if powers.contains(x) {
            map_contains(n.range, g.pow, x)
            let k: Nat satisfy {
                n.range.contains(k) and g.pow(k) = x
            }
            cyclic_subgroup_contains_power(g, k)
            finite_subgroup_contains_eq(s, g.pow(k))
            s.elements.contains(x)
        }
    }
    unique_is_smallest_containing_list(powers, s.elements)
    powers.unique.length <= s.elements.length
    s.elements.length = s.order
    powers.unique.length <= s.order
    n = s.order.suc
    lt_suc(s.order)
    s.order < n
    lte_and_lt(powers.unique.length, s.order, n)
    powers.unique.length < n
    range_pigeonhole(n, g.pow)
    let (i: Nat, j: Nat) satisfy {
        i < j and j < n and g.pow(i) = g.pow(j)
    }
    lt_diff(i, j)
    let d: Nat satisfy { i + d = j and d != Nat.0 }
    add_imp_sub(i, d, j)
    j - i = d
    d <= j
    lte_and_lt(d, j, n)
    d < n
    n = s.order.suc
    lt_suc_right(d, s.order)
    if d = s.order {
        d <= s.order
    }
    if d < s.order {
        d <= s.order
    }
    d <= s.order
    g.pow(j) = g.pow(i)
    pow_cancel(g, i, j)
    g.pow(j - i) = G.1
    g.pow(d) = G.1
    not_lt_zero(d)
    trichotomy(Nat.0, d)
    d > Nat.0 = Nat.0 < d
    d > Nat.0
    cyclic_subgroup_order_le_period(g, d)
    s.order <= d
    lte_antisymm(s.order, d)
    s.order = d
    g.pow(s.order) = G.1
}

/// Powers of an element repeat modulo the order of its cyclic finite subgroup.
theorem pow_mod_cyclic_subgroup_order[G: FiniteGroup](g: G, k: Nat) {
    g.pow(k.mod(cyclic_subgroup_of(g).order)) = g.pow(k)
} by {
    cyclic_subgroup_order_pos(g)
    pow_cyclic_subgroup_order_eq_identity(g)
    pow_mod_order(g, cyclic_subgroup_of(g).order, k)
}

/// The inverse of an element is the preceding power in its cyclic finite period.
theorem inverse_eq_power_cyclic_subgroup_order_pred[G: FiniteGroup](g: G) {
    g.inverse = g.pow(cyclic_subgroup_of(g).order - Nat.1)
} by {
    cyclic_subgroup_order_pos(g)
    pow_cyclic_subgroup_order_eq_identity(g)
    inverse_is_power(g, cyclic_subgroup_of(g).order)
}

/// Adding the cyclic finite subgroup order to an exponent leaves the power unchanged.
theorem pow_add_cyclic_subgroup_order[G: FiniteGroup](g: G, k: Nat) {
    g.pow(k + cyclic_subgroup_of(g).order) = g.pow(k)
} by {
    let m: Nat = cyclic_subgroup_of(g).order
    pow_cyclic_subgroup_order_eq_identity(g)
    g.pow(k) * g.pow(m) = g.pow(k + m)
    g.pow(m) = G.1
    g.pow(k) * G.1 = g.pow(k)
    g.pow(k + m) = g.pow(k)
}

/// Every power of an element belongs to the underlying set of its cyclic finite subgroup.
theorem cyclic_subgroup_as_set_contains_power[G: FiniteGroup](g: G, k: Nat) {
    cyclic_subgroup_of(g).as_set.contains(g.pow(k))
} by {
    cyclic_subgroup_contains_power(g, k)
    cyclic_subgroup_of(g).as_subgroup.contains(g.pow(k))
    cyclic_subgroup_of(g).as_set.contains(g.pow(k))
}

/// Every power of an element belongs to the finite set of its cyclic finite subgroup.
theorem cyclic_subgroup_as_finite_set_contains_power[G: FiniteGroup](g: G, k: Nat) {
    cyclic_subgroup_of(g).as_finite_set.contains(g.pow(k))
} by {
    cyclic_subgroup_elements_eq(g)
    cyclic_powers_contains(g, k)
    cyclic_subgroup_of(g).elements.contains(g.pow(k))
    finite_subgroup_as_finite_set_contains_eq(cyclic_subgroup_of(g), g.pow(k))
    cyclic_subgroup_of(g).as_finite_set.contains(g.pow(k))
}

/// An element belongs to its cyclic finite subgroup.
theorem cyclic_subgroup_contains_generator[G: FiniteGroup](g: G) {
    cyclic_subgroup_of(g).contains(g)
} by {
    cyclic_subgroup_contains_power(g, Nat.1)
    g.pow(Nat.1) = g
    cyclic_subgroup_of(g).contains(g)
}

/// The underlying set of the cyclic finite subgroup contains its generator.
theorem cyclic_subgroup_as_set_contains_generator[G: FiniteGroup](g: G) {
    cyclic_subgroup_of(g).as_set.contains(g)
} by {
    cyclic_subgroup_as_set_contains_power(g, Nat.1)
    g.pow(Nat.1) = g
    cyclic_subgroup_of(g).as_set.contains(g)
}

/// The finite set of the cyclic finite subgroup contains its generator.
theorem cyclic_subgroup_as_finite_set_contains_generator[G: FiniteGroup](g: G) {
    cyclic_subgroup_of(g).as_finite_set.contains(g)
} by {
    cyclic_subgroup_as_finite_set_contains_power(g, Nat.1)
    g.pow(Nat.1) = g
    cyclic_subgroup_of(g).as_finite_set.contains(g)
}

/// The subgroup associated to a cyclic finite subgroup is finitely generated.
theorem cyclic_subgroup_as_subgroup_is_finitely_generated[G: FiniteGroup](g: G) {
    subgroup_is_finitely_generated(cyclic_subgroup_of(g).as_subgroup)
} by {
    subgroup_of_finite_group_is_finitely_generated(cyclic_subgroup_of(g).as_subgroup)
}

attributes G: FiniteGroup {
    /// The trivial subgroup containing only the identity element.
    let identity_subgroup: FiniteSubgroup[G] = isg[G]

    /// The cyclic subgroup generated by this element.
    // TODO: prove this actually is the subgroup you'd expect, not a degenerate case.
    define cyclic_subgroup(self) -> FiniteSubgroup[G] {
        cyclic_subgroup_of(self)
    }
}
