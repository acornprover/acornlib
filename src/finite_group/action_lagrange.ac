from finite_group.base import FiniteGroup, FiniteSubgroup, finite_subgroup_contains_as_subgroup_eq,
    finite_subgroup_as_set_contains_finite_subgroup_eq, cyclic_subgroup_of
from finite_group.action_counting import finite_left_coset_set, finite_left_coset_set_contains_eq,
    finite_left_coset_set_contains_representative, finite_left_coset_set_eq_of_contains_quotient,
    finite_left_coset_representatives, finite_left_coset_representatives_contains_coset,
    same_left_coset_contains_symmetric, same_left_coset_contains_transitive
from finite_group.coset_cardinality import finite_subgroup_left_mul_image_cardinality_is_order
from algebra.group import Group
from algebra.subgroup import Subgroup
from data.basic.set import Set, set_image, set_image_contains_witness,
    image_contains, set_image_contains_eq, set_ext, set_eq_transport_predicate,
    set_eq_universal_of_forall_contains, cardinality_is_well_defined,
    list_set, list_set_contains_eq, unique_list_set_cardinality_is_length
from data.basic.set_list_partition import set_list_union, set_list_union_contains_of_member,
    set_list_all_cardinality_is, set_list_pairwise_disjoint,
    set_list_union_cardinality_is_of_unique_pairwise
from list import List, map, map_contains, unique_list_is_unique, unique_preserves_contains,
    unique_contains_imp_contains
from nat import Nat, divides_mul, divides_self

lemma left_coset_elem_has_subgroup_preimage[G: Group](s: Subgroup[G], a: G, y: G) {
    s.contains(a.inverse * y) implies exists(h: G) { s.contains(h) and a * h = y }
} by {
    if s.contains(a.inverse * y) {
        (a * a.inverse) * y = G.1 * y
        exists(h: G) { s.contains(h) and a * h = y }
    }
}

lemma left_mul_subgroup_into_left_coset[G: Group](s: Subgroup[G], a: G, h: G) {
    s.contains(h) implies s.contains(a.inverse * (a * h))
} by {
    if s.contains(h) {
        s.contains(a.inverse * (a * h))
    }
}

lemma finite_left_mul_image_member_implies_left_coset_set_member[G: FiniteGroup](s: FiniteSubgroup[G], a: G, y: G) {
    set_image[G, G](s.as_set, function(h: G) { a * h }).contains(y) implies finite_left_coset_set(s, a).contains(y)
} by {
    if set_image[G, G](s.as_set, function(h: G) { a * h }).contains(y) {
        set_image_contains_witness[G, G](s.as_set, function(h: G) { a * h }, y)
        let h: G satisfy { s.as_set.contains(h) and y = a * h }
        finite_subgroup_as_set_contains_finite_subgroup_eq[G](s, h)
        finite_subgroup_contains_as_subgroup_eq[G](s, h)
        left_mul_subgroup_into_left_coset[G](s.as_subgroup, a, h)
        finite_subgroup_contains_as_subgroup_eq[G](s, a.inverse * y)
        finite_left_coset_set_contains_eq(s, a, y)
        finite_left_coset_set(s, a).contains(y)
    }
}

lemma finite_left_coset_set_member_implies_left_mul_image_member[G: FiniteGroup](s: FiniteSubgroup[G], a: G, y: G) {
    finite_left_coset_set(s, a).contains(y) implies set_image[G, G](s.as_set, function(h: G) { a * h }).contains(y)
} by {
    if finite_left_coset_set(s, a).contains(y) {
        finite_left_coset_set_contains_eq(s, a, y)
        finite_subgroup_contains_as_subgroup_eq[G](s, a.inverse * y)
        left_coset_elem_has_subgroup_preimage[G](s.as_subgroup, a, y)
        let h: G satisfy { s.as_subgroup.contains(h) and a * h = y }
        finite_subgroup_contains_as_subgroup_eq[G](s, h)
        finite_subgroup_as_set_contains_finite_subgroup_eq[G](s, h)
        image_contains[G, G](function(k: G) { a * k }, s.as_set, y)
        set_image_contains_eq[G, G](s.as_set, function(k: G) { a * k }, y)
        set_image[G, G](s.as_set, function(k: G) { a * k }).contains(y)
    }
}

lemma finite_left_mul_image_contains_eq_left_coset_set_contains[G: FiniteGroup](s: FiniteSubgroup[G], a: G, y: G) {
    set_image[G, G](s.as_set, function(h: G) { a * h }).contains(y) = finite_left_coset_set(s, a).contains(y)
} by {
    if set_image[G, G](s.as_set, function(h: G) { a * h }).contains(y) {
        finite_left_mul_image_member_implies_left_coset_set_member(s, a, y)
    }
    if finite_left_coset_set(s, a).contains(y) {
        finite_left_coset_set_member_implies_left_mul_image_member(s, a, y)
    }
}

lemma finite_subgroup_left_mul_image_eq_left_coset_set[G: FiniteGroup](s: FiniteSubgroup[G], a: G) {
    set_image[G, G](s.as_set, function(h: G) { a * h }) = finite_left_coset_set(s, a)
} by {
    forall(y: G) {
        finite_left_mul_image_contains_eq_left_coset_set_contains(s, a, y)
    }
    set_ext(set_image[G, G](s.as_set, function(h: G) { a * h }), finite_left_coset_set(s, a))
}

/// A finite left coset has the cardinality of its finite subgroup.
theorem finite_left_coset_set_cardinality_is_order[G: FiniteGroup](s: FiniteSubgroup[G], a: G) {
    finite_left_coset_set(s, a).cardinality_is(s.order)
} by {
    let image: Set[G] = set_image[G, G](s.as_set, function(h: G) { a * h })
    let coset: Set[G] = finite_left_coset_set(s, a)
    let p: Set[G] -> Bool = function(t: Set[G]) { t.cardinality_is(s.order) }
    finite_subgroup_left_mul_image_cardinality_is_order(s, a)
    finite_subgroup_left_mul_image_eq_left_coset_set(s, a)
    set_eq_transport_predicate[G](p, image, coset)
}

lemma finite_left_coset_sets_intersecting_eq[G: FiniteGroup](s: FiniteSubgroup[G], a: G, b: G) {
    exists(y: G) { finite_left_coset_set(s, a).contains(y) and finite_left_coset_set(s, b).contains(y) }
    implies finite_left_coset_set(s, a) = finite_left_coset_set(s, b)
} by {
    if exists(y: G) { finite_left_coset_set(s, a).contains(y) and finite_left_coset_set(s, b).contains(y) } {
        let y: G satisfy {
            finite_left_coset_set(s, a).contains(y) and finite_left_coset_set(s, b).contains(y)
        }
        finite_left_coset_set_contains_eq(s, a, y)
        finite_left_coset_set_contains_eq(s, b, y)
        finite_subgroup_contains_as_subgroup_eq(s, a.inverse * y)
        finite_subgroup_contains_as_subgroup_eq(s, b.inverse * y)
        same_left_coset_contains_symmetric(s.as_subgroup, b, y)
        same_left_coset_contains_transitive(s.as_subgroup, a, y, b)
        finite_subgroup_contains_as_subgroup_eq(s, a.inverse * b)
        finite_left_coset_set_eq_of_contains_quotient(s, a, b)
        finite_left_coset_set(s, a) = finite_left_coset_set(s, b)
    }
}

/// Any two finite left cosets of a finite subgroup are equal or disjoint.
theorem finite_left_cosets_equal_or_disjoint[G: FiniteGroup](s: FiniteSubgroup[G], a: G, b: G) {
    finite_left_coset_set(s, a) = finite_left_coset_set(s, b) or
    finite_left_coset_set(s, a).is_disjoint(finite_left_coset_set(s, b))
} by {
    if finite_left_coset_set(s, a) = finite_left_coset_set(s, b) {
    } else {
        forall(y: G) {
            if finite_left_coset_set(s, a).contains(y) and finite_left_coset_set(s, b).contains(y) {
                finite_left_coset_sets_intersecting_eq(s, a, b)
                false
            }
            not (finite_left_coset_set(s, a).contains(y) and finite_left_coset_set(s, b).contains(y))
        }
        finite_left_coset_set(s, a).is_disjoint(finite_left_coset_set(s, b))
    }
}

/// The finite left-coset representative list is duplicate-free.
theorem finite_left_coset_representatives_is_unique[G: FiniteGroup](s: FiniteSubgroup[G]) {
    finite_left_coset_representatives(s).is_unique
} by {
    let raw: List[Set[G]] = map[G, Set[G]](G.elements, function(a: G) { finite_left_coset_set(s, a) })
    unique_list_is_unique[Set[G]](raw)
}

/// Every listed finite left coset has cardinality equal to the subgroup order.
theorem finite_left_coset_representative_cardinality_is_order[G: FiniteGroup](s: FiniteSubgroup[G], c: Set[G]) {
    finite_left_coset_representatives(s).contains(c) implies c.cardinality_is(s.order)
} by {
    if finite_left_coset_representatives(s).contains(c) {
        let raw: List[Set[G]] = map[G, Set[G]](G.elements, function(a: G) { finite_left_coset_set(s, a) })
        unique_preserves_contains[Set[G]](raw, c)
        map_contains[G, Set[G]](G.elements, function(a: G) { finite_left_coset_set(s, a) }, c)
        let a: G satisfy {
            G.elements.contains(a) and finite_left_coset_set(s, a) = c
        }
        finite_left_coset_set_cardinality_is_order(s, a)
        c.cardinality_is(s.order)
    }
}

lemma finite_left_coset_representatives_distinct_disjoint[G: FiniteGroup](s: FiniteSubgroup[G], c: Set[G], d: Set[G]) {
    finite_left_coset_representatives(s).contains(c) and
    finite_left_coset_representatives(s).contains(d) and
    c != d
    implies c.is_disjoint(d)
} by {
    if finite_left_coset_representatives(s).contains(c) and
        finite_left_coset_representatives(s).contains(d) and c != d {
        let raw: List[Set[G]] = map[G, Set[G]](G.elements, function(a: G) { finite_left_coset_set(s, a) })
        raw.unique.contains(c)
        raw.unique.contains(d)
        unique_contains_imp_contains[Set[G]](raw, c)
        unique_contains_imp_contains[Set[G]](raw, d)
        map_contains[G, Set[G]](G.elements, function(a: G) { finite_left_coset_set(s, a) }, c)
        map_contains[G, Set[G]](G.elements, function(a: G) { finite_left_coset_set(s, a) }, d)
        let a: G satisfy { G.elements.contains(a) and finite_left_coset_set(s, a) = c }
        let b: G satisfy { G.elements.contains(b) and finite_left_coset_set(s, b) = d }
        finite_left_cosets_equal_or_disjoint(s, a, b)
        if finite_left_coset_set(s, a) = finite_left_coset_set(s, b) {
            false
        }
        if finite_left_coset_set(s, a).is_disjoint(finite_left_coset_set(s, b)) {
            c.is_disjoint(d)
        }
        c.is_disjoint(d)
    }
}

/// The recursive union of finite left-coset representatives is the universal set.
theorem finite_left_coset_representatives_union_covers_universal[G: FiniteGroup](s: FiniteSubgroup[G]) {
    set_list_union[G](finite_left_coset_representatives(s)) = Set[G].universal_set
} by {
    forall(x: G) {
        finite_left_coset_representatives_contains_coset(s, x)
        finite_left_coset_set_contains_representative(s, x)
        set_list_union_contains_of_member[G](finite_left_coset_representatives(s), finite_left_coset_set(s, x), x)
        set_list_union[G](finite_left_coset_representatives(s)).contains(x)
    }
    set_eq_universal_of_forall_contains[G](set_list_union[G](finite_left_coset_representatives(s)))
}

/// The universal set of a finite group has cardinality equal to the group order.
theorem finite_group_universal_set_cardinality_is_order[G: FiniteGroup] {
    Set[G].universal_set.cardinality_is(G.order)
} by {
    forall(x: G) {
        G.elements.contains(x)
        list_set_contains_eq[G](G.elements, x)
        list_set[G](G.elements).contains(x)
    }
    set_eq_universal_of_forall_contains[G](list_set[G](G.elements))
    list_set[G](G.elements) = Set[G].universal_set
    unique_list_set_cardinality_is_length[G](G.elements)
    list_set[G](G.elements).cardinality_is(G.elements.length)
}

/// Every finite left-coset representative has cardinality equal to the subgroup order.
theorem finite_left_coset_representatives_all_cardinality_is_order[G: FiniteGroup](s: FiniteSubgroup[G]) {
    set_list_all_cardinality_is[G](finite_left_coset_representatives(s), s.order)
} by {
    forall(c: Set[G]) {
        if finite_left_coset_representatives(s).contains(c) {
            finite_left_coset_representative_cardinality_is_order(s, c)
            c.cardinality_is(s.order)
        }
        finite_left_coset_representatives(s).contains(c) implies c.cardinality_is(s.order)
    }
}

/// Distinct finite left-coset representatives are pairwise disjoint.
theorem finite_left_coset_representatives_pairwise_disjoint[G: FiniteGroup](s: FiniteSubgroup[G]) {
    set_list_pairwise_disjoint[G](finite_left_coset_representatives(s))
} by {
    forall(c: Set[G], d: Set[G]) {
        if finite_left_coset_representatives(s).contains(c) and
            finite_left_coset_representatives(s).contains(d) and c != d {
            finite_left_coset_representatives_distinct_disjoint(s, c, d)
            c.is_disjoint(d)
        }
    }
}

/// The union of the finite left-coset representatives has cardinality `s.order * reps.length`.
theorem finite_left_coset_representatives_union_cardinality[G: FiniteGroup](s: FiniteSubgroup[G]) {
    set_list_union[G](finite_left_coset_representatives(s)).cardinality_is(
        s.order * finite_left_coset_representatives(s).length)
} by {
    finite_left_coset_representatives_is_unique(s)
    finite_left_coset_representatives_pairwise_disjoint(s)
    finite_left_coset_representatives_all_cardinality_is_order(s)
    set_list_union_cardinality_is_of_unique_pairwise[G](finite_left_coset_representatives(s), s.order)
}

/// The subgroup order times the number of finite left cosets equals the group order.
theorem finite_left_coset_representatives_order_product_eq_group_order[G: FiniteGroup](s: FiniteSubgroup[G]) {
    s.order * finite_left_coset_representatives(s).length = G.order
} by {
    let u: Set[G] = set_list_union[G](finite_left_coset_representatives(s))
    let n1: Nat = s.order * finite_left_coset_representatives(s).length
    let n2: Nat = G.order
    finite_left_coset_representatives_union_cardinality(s)
    finite_left_coset_representatives_union_covers_universal(s)
    finite_group_universal_set_cardinality_is_order[G]
    u.cardinality_is(n1)
    u.cardinality_is(n2)
    cardinality_is_well_defined[G](u, n1, n2)
}

/// Lagrange's theorem in explicit-witness form: the subgroup order times the
/// number of finite left cosets is the group order.
theorem finite_subgroup_order_divides_group_order_witness[G: FiniteGroup](s: FiniteSubgroup[G]) {
    exists(k: Nat) { s.order * k = G.order }
} by {
    let reps_len: Nat = finite_left_coset_representatives(s).length
    finite_left_coset_representatives_order_product_eq_group_order(s)
}

/// Lagrange's theorem: the order of a finite subgroup divides the group order.
theorem finite_subgroup_order_divides_group_order[G: FiniteGroup](s: FiniteSubgroup[G]) {
    s.order.divides(G.order)
} by {
    let reps_len: Nat = finite_left_coset_representatives(s).length
    divides_self(s.order)
    divides_mul(s.order, s.order, reps_len)
    finite_left_coset_representatives_order_product_eq_group_order(s)
}

/// The order of the finite cyclic subgroup generated by an element divides the group order.
theorem finite_cyclic_subgroup_order_divides_group_order[G: FiniteGroup](g: G) {
    cyclic_subgroup_of(g).order.divides(G.order)
} by {
    finite_subgroup_order_divides_group_order(cyclic_subgroup_of(g))
}
