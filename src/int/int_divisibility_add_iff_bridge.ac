from int.lattice import Int
from int.int_divisibility import int_divides_add
from int.int_divisibility_extra import int_divides_add_cancel_left,
    int_divides_add_cancel_right
from data.basic.logic import iff_bool_imp_eq

/// If `a | b`, divisibility of `b + c` is equivalent to divisibility of `c`.
theorem int_divides_add_left_iff(a: Int, b: Int, c: Int) {
    a.divides(b) implies (a.divides(b + c) = a.divides(c))
} by {
    if a.divides(b) {
        if a.divides(b + c) {
            int_divides_add_cancel_left(a, b, c)
            a.divides(c)
        }
        if a.divides(c) {
            int_divides_add(a, b, c)
            a.divides(b + c)
        }
        (a.divides(b + c) implies a.divides(c))
        (a.divides(c) implies a.divides(b + c))
        iff_bool_imp_eq(a.divides(b + c), a.divides(c))
        a.divides(b + c) = a.divides(c)
    }
}

/// If `a | c`, divisibility of `b + c` is equivalent to divisibility of `b`.
theorem int_divides_add_right_iff(a: Int, b: Int, c: Int) {
    a.divides(c) implies (a.divides(b + c) = a.divides(b))
} by {
    if a.divides(c) {
        if a.divides(b + c) {
            int_divides_add_cancel_right(a, b, c)
            a.divides(b)
        }
        if a.divides(b) {
            int_divides_add(a, b, c)
            a.divides(b + c)
        }
        (a.divides(b + c) implies a.divides(b))
        (a.divides(b) implies a.divides(b + c))
        iff_bool_imp_eq(a.divides(b + c), a.divides(b))
        a.divides(b + c) = a.divides(b)
    }
}

/// Replacing a divisible left summand preserves divisibility of the sum.
theorem int_divides_add_left_replace_iff(a: Int, b: Int, c: Int, d: Int) {
    a.divides(b) and a.divides(c) implies
        (a.divides(b + d) = a.divides(c + d))
} by {
    if a.divides(b) and a.divides(c) {
        int_divides_add_left_iff(a, b, d)
        a.divides(b + d) = a.divides(d)
        int_divides_add_left_iff(a, c, d)
        a.divides(c + d) = a.divides(d)
        a.divides(b + d) = a.divides(c + d)
    }
}

/// Replacing a divisible right summand preserves divisibility of the sum.
theorem int_divides_add_right_replace_iff(a: Int, b: Int, c: Int, d: Int) {
    a.divides(c) and a.divides(d) implies
        (a.divides(b + c) = a.divides(b + d))
} by {
    if a.divides(c) and a.divides(d) {
        int_divides_add_right_iff(a, b, c)
        a.divides(b + c) = a.divides(b)
        int_divides_add_right_iff(a, b, d)
        a.divides(b + d) = a.divides(b)
        a.divides(b + c) = a.divides(b + d)
    }
}
