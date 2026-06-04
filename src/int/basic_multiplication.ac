from int.int_base import Int
from int.int_arith import mul_neg_from_nat_left, mul_neg_from_nat_right, mul_neg_from_nat_neg
from int.int_base import mul_zero_left, mul_zero_right
from int.lattice import mul_from_nat
from nat import Nat

numerals Nat

/// Positive integer multiplication facts for embedded natural numerals from 1 through 9.
theorem int_mul_pos_1_1 {
    Int.from_nat(1) * Int.from_nat(1) = Int.from_nat(1)
} by {
    mul_from_nat(1, 1)
    lib(nat).nat_mul_1_1
}

theorem int_mul_pos_1_2 {
    Int.from_nat(1) * Int.from_nat(2) = Int.from_nat(2)
} by {
    mul_from_nat(1, 2)
    lib(nat).nat_mul_1_2
}

theorem int_mul_pos_1_3 {
    Int.from_nat(1) * Int.from_nat(3) = Int.from_nat(3)
} by {
    mul_from_nat(1, 3)
    lib(nat).nat_mul_1_3
}

theorem int_mul_pos_1_4 {
    Int.from_nat(1) * Int.from_nat(4) = Int.from_nat(4)
} by {
    mul_from_nat(1, 4)
    lib(nat).nat_mul_1_4
}

theorem int_mul_pos_1_5 {
    Int.from_nat(1) * Int.from_nat(5) = Int.from_nat(5)
} by {
    mul_from_nat(1, 5)
    lib(nat).nat_mul_1_5
}

theorem int_mul_pos_1_6 {
    Int.from_nat(1) * Int.from_nat(6) = Int.from_nat(6)
} by {
    mul_from_nat(1, 6)
    lib(nat).nat_mul_1_6
}

theorem int_mul_pos_1_7 {
    Int.from_nat(1) * Int.from_nat(7) = Int.from_nat(7)
} by {
    mul_from_nat(1, 7)
    lib(nat).nat_mul_1_7
}

theorem int_mul_pos_1_8 {
    Int.from_nat(1) * Int.from_nat(8) = Int.from_nat(8)
} by {
    mul_from_nat(1, 8)
    lib(nat).nat_mul_1_8
}

theorem int_mul_pos_1_9 {
    Int.from_nat(1) * Int.from_nat(9) = Int.from_nat(9)
} by {
    mul_from_nat(1, 9)
    lib(nat).nat_mul_1_9
}

theorem int_mul_pos_2_1 {
    Int.from_nat(2) * Int.from_nat(1) = Int.from_nat(2)
} by {
    mul_from_nat(2, 1)
    lib(nat).nat_mul_2_1
}

theorem int_mul_pos_2_2 {
    Int.from_nat(2) * Int.from_nat(2) = Int.from_nat(4)
} by {
    mul_from_nat(2, 2)
    lib(nat).nat_mul_2_2
}

theorem int_mul_pos_2_3 {
    Int.from_nat(2) * Int.from_nat(3) = Int.from_nat(6)
} by {
    mul_from_nat(2, 3)
    lib(nat).nat_mul_2_3
}

theorem int_mul_pos_2_4 {
    Int.from_nat(2) * Int.from_nat(4) = Int.from_nat(8)
} by {
    mul_from_nat(2, 4)
    lib(nat).nat_mul_2_4
}

theorem int_mul_pos_2_5 {
    Int.from_nat(2) * Int.from_nat(5) = Int.from_nat(10)
} by {
    mul_from_nat(2, 5)
    lib(nat).nat_mul_2_5
}

theorem int_mul_pos_2_6 {
    Int.from_nat(2) * Int.from_nat(6) = Int.from_nat(12)
} by {
    mul_from_nat(2, 6)
    lib(nat).nat_mul_2_6
}

theorem int_mul_pos_2_7 {
    Int.from_nat(2) * Int.from_nat(7) = Int.from_nat(14)
} by {
    mul_from_nat(2, 7)
    lib(nat).nat_mul_2_7
}

theorem int_mul_pos_2_8 {
    Int.from_nat(2) * Int.from_nat(8) = Int.from_nat(16)
} by {
    mul_from_nat(2, 8)
    lib(nat).nat_mul_2_8
}

theorem int_mul_pos_2_9 {
    Int.from_nat(2) * Int.from_nat(9) = Int.from_nat(18)
} by {
    mul_from_nat(2, 9)
    lib(nat).nat_mul_2_9
}

theorem int_mul_pos_3_1 {
    Int.from_nat(3) * Int.from_nat(1) = Int.from_nat(3)
} by {
    mul_from_nat(3, 1)
    lib(nat).nat_mul_3_1
}

theorem int_mul_pos_3_2 {
    Int.from_nat(3) * Int.from_nat(2) = Int.from_nat(6)
} by {
    mul_from_nat(3, 2)
    lib(nat).nat_mul_3_2
}

theorem int_mul_pos_3_3 {
    Int.from_nat(3) * Int.from_nat(3) = Int.from_nat(9)
} by {
    mul_from_nat(3, 3)
    lib(nat).nat_mul_3_3
}

theorem int_mul_pos_3_4 {
    Int.from_nat(3) * Int.from_nat(4) = Int.from_nat(12)
} by {
    mul_from_nat(3, 4)
    lib(nat).nat_mul_3_4
}

theorem int_mul_pos_3_5 {
    Int.from_nat(3) * Int.from_nat(5) = Int.from_nat(15)
} by {
    mul_from_nat(3, 5)
    lib(nat).nat_mul_3_5
}

theorem int_mul_pos_3_6 {
    Int.from_nat(3) * Int.from_nat(6) = Int.from_nat(18)
} by {
    mul_from_nat(3, 6)
    lib(nat).nat_mul_3_6
}

theorem int_mul_pos_3_7 {
    Int.from_nat(3) * Int.from_nat(7) = Int.from_nat(21)
} by {
    mul_from_nat(3, 7)
    lib(nat).nat_mul_3_7
}

theorem int_mul_pos_3_8 {
    Int.from_nat(3) * Int.from_nat(8) = Int.from_nat(24)
} by {
    mul_from_nat(3, 8)
    lib(nat).nat_mul_3_8
}

theorem int_mul_pos_3_9 {
    Int.from_nat(3) * Int.from_nat(9) = Int.from_nat(27)
} by {
    mul_from_nat(3, 9)
    lib(nat).nat_mul_3_9
}

theorem int_mul_pos_4_1 {
    Int.from_nat(4) * Int.from_nat(1) = Int.from_nat(4)
} by {
    mul_from_nat(4, 1)
    lib(nat).nat_mul_4_1
}

theorem int_mul_pos_4_2 {
    Int.from_nat(4) * Int.from_nat(2) = Int.from_nat(8)
} by {
    mul_from_nat(4, 2)
    lib(nat).nat_mul_4_2
}

theorem int_mul_pos_4_3 {
    Int.from_nat(4) * Int.from_nat(3) = Int.from_nat(12)
} by {
    mul_from_nat(4, 3)
    lib(nat).nat_mul_4_3
}

theorem int_mul_pos_4_4 {
    Int.from_nat(4) * Int.from_nat(4) = Int.from_nat(16)
} by {
    mul_from_nat(4, 4)
    lib(nat).nat_mul_4_4
}

theorem int_mul_pos_4_5 {
    Int.from_nat(4) * Int.from_nat(5) = Int.from_nat(20)
} by {
    mul_from_nat(4, 5)
    lib(nat).nat_mul_4_5
}

theorem int_mul_pos_4_6 {
    Int.from_nat(4) * Int.from_nat(6) = Int.from_nat(24)
} by {
    mul_from_nat(4, 6)
    lib(nat).nat_mul_4_6
}

theorem int_mul_pos_4_7 {
    Int.from_nat(4) * Int.from_nat(7) = Int.from_nat(28)
} by {
    mul_from_nat(4, 7)
    lib(nat).nat_mul_4_7
}

theorem int_mul_pos_4_8 {
    Int.from_nat(4) * Int.from_nat(8) = Int.from_nat(32)
} by {
    mul_from_nat(4, 8)
    lib(nat).nat_mul_4_8
}

theorem int_mul_pos_4_9 {
    Int.from_nat(4) * Int.from_nat(9) = Int.from_nat(36)
} by {
    mul_from_nat(4, 9)
    lib(nat).nat_mul_4_9
}

theorem int_mul_pos_5_1 {
    Int.from_nat(5) * Int.from_nat(1) = Int.from_nat(5)
} by {
    mul_from_nat(5, 1)
    lib(nat).nat_mul_5_1
}

theorem int_mul_pos_5_2 {
    Int.from_nat(5) * Int.from_nat(2) = Int.from_nat(10)
} by {
    mul_from_nat(5, 2)
    lib(nat).nat_mul_5_2
}

theorem int_mul_pos_5_3 {
    Int.from_nat(5) * Int.from_nat(3) = Int.from_nat(15)
} by {
    mul_from_nat(5, 3)
    lib(nat).nat_mul_5_3
}

theorem int_mul_pos_5_4 {
    Int.from_nat(5) * Int.from_nat(4) = Int.from_nat(20)
} by {
    mul_from_nat(5, 4)
    lib(nat).nat_mul_5_4
}

theorem int_mul_pos_5_5 {
    Int.from_nat(5) * Int.from_nat(5) = Int.from_nat(25)
} by {
    mul_from_nat(5, 5)
    lib(nat).nat_mul_5_5
}

theorem int_mul_pos_5_6 {
    Int.from_nat(5) * Int.from_nat(6) = Int.from_nat(30)
} by {
    mul_from_nat(5, 6)
    lib(nat).nat_mul_5_6
}

theorem int_mul_pos_5_7 {
    Int.from_nat(5) * Int.from_nat(7) = Int.from_nat(35)
} by {
    mul_from_nat(5, 7)
    lib(nat).nat_mul_5_7
}

theorem int_mul_pos_5_8 {
    Int.from_nat(5) * Int.from_nat(8) = Int.from_nat(40)
} by {
    mul_from_nat(5, 8)
    lib(nat).nat_mul_5_8
}

theorem int_mul_pos_5_9 {
    Int.from_nat(5) * Int.from_nat(9) = Int.from_nat(45)
} by {
    mul_from_nat(5, 9)
    lib(nat).nat_mul_5_9
}

theorem int_mul_pos_6_1 {
    Int.from_nat(6) * Int.from_nat(1) = Int.from_nat(6)
} by {
    mul_from_nat(6, 1)
    lib(nat).nat_mul_6_1
}

theorem int_mul_pos_6_2 {
    Int.from_nat(6) * Int.from_nat(2) = Int.from_nat(12)
} by {
    mul_from_nat(6, 2)
    lib(nat).nat_mul_6_2
}

theorem int_mul_pos_6_3 {
    Int.from_nat(6) * Int.from_nat(3) = Int.from_nat(18)
} by {
    mul_from_nat(6, 3)
    lib(nat).nat_mul_6_3
}

theorem int_mul_pos_6_4 {
    Int.from_nat(6) * Int.from_nat(4) = Int.from_nat(24)
} by {
    mul_from_nat(6, 4)
    lib(nat).nat_mul_6_4
}

theorem int_mul_pos_6_5 {
    Int.from_nat(6) * Int.from_nat(5) = Int.from_nat(30)
} by {
    mul_from_nat(6, 5)
    lib(nat).nat_mul_6_5
}

theorem int_mul_pos_6_6 {
    Int.from_nat(6) * Int.from_nat(6) = Int.from_nat(36)
} by {
    mul_from_nat(6, 6)
    lib(nat).nat_mul_6_6
}

theorem int_mul_pos_6_7 {
    Int.from_nat(6) * Int.from_nat(7) = Int.from_nat(42)
} by {
    mul_from_nat(6, 7)
    lib(nat).nat_mul_6_7
}

theorem int_mul_pos_6_8 {
    Int.from_nat(6) * Int.from_nat(8) = Int.from_nat(48)
} by {
    mul_from_nat(6, 8)
    lib(nat).nat_mul_6_8
}

theorem int_mul_pos_6_9 {
    Int.from_nat(6) * Int.from_nat(9) = Int.from_nat(54)
} by {
    mul_from_nat(6, 9)
    lib(nat).nat_mul_6_9
}

theorem int_mul_pos_7_1 {
    Int.from_nat(7) * Int.from_nat(1) = Int.from_nat(7)
} by {
    mul_from_nat(7, 1)
    lib(nat).nat_mul_7_1
}

theorem int_mul_pos_7_2 {
    Int.from_nat(7) * Int.from_nat(2) = Int.from_nat(14)
} by {
    mul_from_nat(7, 2)
    lib(nat).nat_mul_7_2
}

theorem int_mul_pos_7_3 {
    Int.from_nat(7) * Int.from_nat(3) = Int.from_nat(21)
} by {
    mul_from_nat(7, 3)
    lib(nat).nat_mul_7_3
}

theorem int_mul_pos_7_4 {
    Int.from_nat(7) * Int.from_nat(4) = Int.from_nat(28)
} by {
    mul_from_nat(7, 4)
    lib(nat).nat_mul_7_4
}

theorem int_mul_pos_7_5 {
    Int.from_nat(7) * Int.from_nat(5) = Int.from_nat(35)
} by {
    mul_from_nat(7, 5)
    lib(nat).nat_mul_7_5
}

theorem int_mul_pos_7_6 {
    Int.from_nat(7) * Int.from_nat(6) = Int.from_nat(42)
} by {
    mul_from_nat(7, 6)
    lib(nat).nat_mul_7_6
}

theorem int_mul_pos_7_7 {
    Int.from_nat(7) * Int.from_nat(7) = Int.from_nat(49)
} by {
    mul_from_nat(7, 7)
    lib(nat).nat_mul_7_7
}

theorem int_mul_pos_7_8 {
    Int.from_nat(7) * Int.from_nat(8) = Int.from_nat(56)
} by {
    mul_from_nat(7, 8)
    lib(nat).nat_mul_7_8
}

theorem int_mul_pos_7_9 {
    Int.from_nat(7) * Int.from_nat(9) = Int.from_nat(63)
} by {
    mul_from_nat(7, 9)
    lib(nat).nat_mul_7_9
}

theorem int_mul_pos_8_1 {
    Int.from_nat(8) * Int.from_nat(1) = Int.from_nat(8)
} by {
    mul_from_nat(8, 1)
    lib(nat).nat_mul_8_1
}

theorem int_mul_pos_8_2 {
    Int.from_nat(8) * Int.from_nat(2) = Int.from_nat(16)
} by {
    mul_from_nat(8, 2)
    lib(nat).nat_mul_8_2
}

theorem int_mul_pos_8_3 {
    Int.from_nat(8) * Int.from_nat(3) = Int.from_nat(24)
} by {
    mul_from_nat(8, 3)
    lib(nat).nat_mul_8_3
}

theorem int_mul_pos_8_4 {
    Int.from_nat(8) * Int.from_nat(4) = Int.from_nat(32)
} by {
    mul_from_nat(8, 4)
    lib(nat).nat_mul_8_4
}

theorem int_mul_pos_8_5 {
    Int.from_nat(8) * Int.from_nat(5) = Int.from_nat(40)
} by {
    mul_from_nat(8, 5)
    lib(nat).nat_mul_8_5
}

theorem int_mul_pos_8_6 {
    Int.from_nat(8) * Int.from_nat(6) = Int.from_nat(48)
} by {
    mul_from_nat(8, 6)
    lib(nat).nat_mul_8_6
}

theorem int_mul_pos_8_7 {
    Int.from_nat(8) * Int.from_nat(7) = Int.from_nat(56)
} by {
    mul_from_nat(8, 7)
    lib(nat).nat_mul_8_7
}

theorem int_mul_pos_8_8 {
    Int.from_nat(8) * Int.from_nat(8) = Int.from_nat(64)
} by {
    mul_from_nat(8, 8)
    lib(nat).nat_mul_8_8
}

theorem int_mul_pos_8_9 {
    Int.from_nat(8) * Int.from_nat(9) = Int.from_nat(72)
} by {
    mul_from_nat(8, 9)
    lib(nat).nat_mul_8_9
}

theorem int_mul_pos_9_1 {
    Int.from_nat(9) * Int.from_nat(1) = Int.from_nat(9)
} by {
    mul_from_nat(9, 1)
    lib(nat).nat_mul_9_1
}

theorem int_mul_pos_9_2 {
    Int.from_nat(9) * Int.from_nat(2) = Int.from_nat(18)
} by {
    mul_from_nat(9, 2)
    lib(nat).nat_mul_9_2
}

theorem int_mul_pos_9_3 {
    Int.from_nat(9) * Int.from_nat(3) = Int.from_nat(27)
} by {
    mul_from_nat(9, 3)
    lib(nat).nat_mul_9_3
}

theorem int_mul_pos_9_4 {
    Int.from_nat(9) * Int.from_nat(4) = Int.from_nat(36)
} by {
    mul_from_nat(9, 4)
    lib(nat).nat_mul_9_4
}

theorem int_mul_pos_9_5 {
    Int.from_nat(9) * Int.from_nat(5) = Int.from_nat(45)
} by {
    mul_from_nat(9, 5)
    lib(nat).nat_mul_9_5
}

theorem int_mul_pos_9_6 {
    Int.from_nat(9) * Int.from_nat(6) = Int.from_nat(54)
} by {
    mul_from_nat(9, 6)
    lib(nat).nat_mul_9_6
}

theorem int_mul_pos_9_7 {
    Int.from_nat(9) * Int.from_nat(7) = Int.from_nat(63)
} by {
    mul_from_nat(9, 7)
    lib(nat).nat_mul_9_7
}

theorem int_mul_pos_9_8 {
    Int.from_nat(9) * Int.from_nat(8) = Int.from_nat(72)
} by {
    mul_from_nat(9, 8)
    lib(nat).nat_mul_9_8
}

theorem int_mul_pos_9_9 {
    Int.from_nat(9) * Int.from_nat(9) = Int.from_nat(81)
} by {
    mul_from_nat(9, 9)
    lib(nat).nat_mul_9_9
}

/// Negative-left integer multiplication facts for embedded natural numerals from 1 through 9.
theorem int_mul_neg_left_1_1 {
    -Int.from_nat(1) * Int.from_nat(1) = -Int.from_nat(1)
} by {
    mul_neg_from_nat_left(1, 1)
    lib(nat).nat_mul_1_1
}

theorem int_mul_neg_left_1_2 {
    -Int.from_nat(1) * Int.from_nat(2) = -Int.from_nat(2)
} by {
    mul_neg_from_nat_left(1, 2)
    lib(nat).nat_mul_1_2
}

theorem int_mul_neg_left_1_3 {
    -Int.from_nat(1) * Int.from_nat(3) = -Int.from_nat(3)
} by {
    mul_neg_from_nat_left(1, 3)
    lib(nat).nat_mul_1_3
}

theorem int_mul_neg_left_1_4 {
    -Int.from_nat(1) * Int.from_nat(4) = -Int.from_nat(4)
} by {
    mul_neg_from_nat_left(1, 4)
    lib(nat).nat_mul_1_4
}

theorem int_mul_neg_left_1_5 {
    -Int.from_nat(1) * Int.from_nat(5) = -Int.from_nat(5)
} by {
    mul_neg_from_nat_left(1, 5)
    lib(nat).nat_mul_1_5
}

theorem int_mul_neg_left_1_6 {
    -Int.from_nat(1) * Int.from_nat(6) = -Int.from_nat(6)
} by {
    mul_neg_from_nat_left(1, 6)
    lib(nat).nat_mul_1_6
}

theorem int_mul_neg_left_1_7 {
    -Int.from_nat(1) * Int.from_nat(7) = -Int.from_nat(7)
} by {
    mul_neg_from_nat_left(1, 7)
    lib(nat).nat_mul_1_7
}

theorem int_mul_neg_left_1_8 {
    -Int.from_nat(1) * Int.from_nat(8) = -Int.from_nat(8)
} by {
    mul_neg_from_nat_left(1, 8)
    lib(nat).nat_mul_1_8
}

theorem int_mul_neg_left_1_9 {
    -Int.from_nat(1) * Int.from_nat(9) = -Int.from_nat(9)
} by {
    mul_neg_from_nat_left(1, 9)
    lib(nat).nat_mul_1_9
}

theorem int_mul_neg_left_2_1 {
    -Int.from_nat(2) * Int.from_nat(1) = -Int.from_nat(2)
} by {
    mul_neg_from_nat_left(2, 1)
    lib(nat).nat_mul_2_1
}

theorem int_mul_neg_left_2_2 {
    -Int.from_nat(2) * Int.from_nat(2) = -Int.from_nat(4)
} by {
    mul_neg_from_nat_left(2, 2)
    lib(nat).nat_mul_2_2
}

theorem int_mul_neg_left_2_3 {
    -Int.from_nat(2) * Int.from_nat(3) = -Int.from_nat(6)
} by {
    mul_neg_from_nat_left(2, 3)
    lib(nat).nat_mul_2_3
}

theorem int_mul_neg_left_2_4 {
    -Int.from_nat(2) * Int.from_nat(4) = -Int.from_nat(8)
} by {
    mul_neg_from_nat_left(2, 4)
    lib(nat).nat_mul_2_4
}

theorem int_mul_neg_left_2_5 {
    -Int.from_nat(2) * Int.from_nat(5) = -Int.from_nat(10)
} by {
    mul_neg_from_nat_left(2, 5)
    lib(nat).nat_mul_2_5
}

theorem int_mul_neg_left_2_6 {
    -Int.from_nat(2) * Int.from_nat(6) = -Int.from_nat(12)
} by {
    mul_neg_from_nat_left(2, 6)
    lib(nat).nat_mul_2_6
}

theorem int_mul_neg_left_2_7 {
    -Int.from_nat(2) * Int.from_nat(7) = -Int.from_nat(14)
} by {
    mul_neg_from_nat_left(2, 7)
    lib(nat).nat_mul_2_7
}

theorem int_mul_neg_left_2_8 {
    -Int.from_nat(2) * Int.from_nat(8) = -Int.from_nat(16)
} by {
    mul_neg_from_nat_left(2, 8)
    lib(nat).nat_mul_2_8
}

theorem int_mul_neg_left_2_9 {
    -Int.from_nat(2) * Int.from_nat(9) = -Int.from_nat(18)
} by {
    mul_neg_from_nat_left(2, 9)
    lib(nat).nat_mul_2_9
}

theorem int_mul_neg_left_3_1 {
    -Int.from_nat(3) * Int.from_nat(1) = -Int.from_nat(3)
} by {
    mul_neg_from_nat_left(3, 1)
    lib(nat).nat_mul_3_1
}

theorem int_mul_neg_left_3_2 {
    -Int.from_nat(3) * Int.from_nat(2) = -Int.from_nat(6)
} by {
    mul_neg_from_nat_left(3, 2)
    lib(nat).nat_mul_3_2
}

theorem int_mul_neg_left_3_3 {
    -Int.from_nat(3) * Int.from_nat(3) = -Int.from_nat(9)
} by {
    mul_neg_from_nat_left(3, 3)
    lib(nat).nat_mul_3_3
}

theorem int_mul_neg_left_3_4 {
    -Int.from_nat(3) * Int.from_nat(4) = -Int.from_nat(12)
} by {
    mul_neg_from_nat_left(3, 4)
    lib(nat).nat_mul_3_4
}

theorem int_mul_neg_left_3_5 {
    -Int.from_nat(3) * Int.from_nat(5) = -Int.from_nat(15)
} by {
    mul_neg_from_nat_left(3, 5)
    lib(nat).nat_mul_3_5
}

theorem int_mul_neg_left_3_6 {
    -Int.from_nat(3) * Int.from_nat(6) = -Int.from_nat(18)
} by {
    mul_neg_from_nat_left(3, 6)
    lib(nat).nat_mul_3_6
}

theorem int_mul_neg_left_3_7 {
    -Int.from_nat(3) * Int.from_nat(7) = -Int.from_nat(21)
} by {
    mul_neg_from_nat_left(3, 7)
    lib(nat).nat_mul_3_7
}

theorem int_mul_neg_left_3_8 {
    -Int.from_nat(3) * Int.from_nat(8) = -Int.from_nat(24)
} by {
    mul_neg_from_nat_left(3, 8)
    lib(nat).nat_mul_3_8
}

theorem int_mul_neg_left_3_9 {
    -Int.from_nat(3) * Int.from_nat(9) = -Int.from_nat(27)
} by {
    mul_neg_from_nat_left(3, 9)
    lib(nat).nat_mul_3_9
}

theorem int_mul_neg_left_4_1 {
    -Int.from_nat(4) * Int.from_nat(1) = -Int.from_nat(4)
} by {
    mul_neg_from_nat_left(4, 1)
    lib(nat).nat_mul_4_1
}

theorem int_mul_neg_left_4_2 {
    -Int.from_nat(4) * Int.from_nat(2) = -Int.from_nat(8)
} by {
    mul_neg_from_nat_left(4, 2)
    lib(nat).nat_mul_4_2
}

theorem int_mul_neg_left_4_3 {
    -Int.from_nat(4) * Int.from_nat(3) = -Int.from_nat(12)
} by {
    mul_neg_from_nat_left(4, 3)
    lib(nat).nat_mul_4_3
}

theorem int_mul_neg_left_4_4 {
    -Int.from_nat(4) * Int.from_nat(4) = -Int.from_nat(16)
} by {
    mul_neg_from_nat_left(4, 4)
    lib(nat).nat_mul_4_4
}

theorem int_mul_neg_left_4_5 {
    -Int.from_nat(4) * Int.from_nat(5) = -Int.from_nat(20)
} by {
    mul_neg_from_nat_left(4, 5)
    lib(nat).nat_mul_4_5
}

theorem int_mul_neg_left_4_6 {
    -Int.from_nat(4) * Int.from_nat(6) = -Int.from_nat(24)
} by {
    mul_neg_from_nat_left(4, 6)
    lib(nat).nat_mul_4_6
}

theorem int_mul_neg_left_4_7 {
    -Int.from_nat(4) * Int.from_nat(7) = -Int.from_nat(28)
} by {
    mul_neg_from_nat_left(4, 7)
    lib(nat).nat_mul_4_7
}

theorem int_mul_neg_left_4_8 {
    -Int.from_nat(4) * Int.from_nat(8) = -Int.from_nat(32)
} by {
    mul_neg_from_nat_left(4, 8)
    lib(nat).nat_mul_4_8
}

theorem int_mul_neg_left_4_9 {
    -Int.from_nat(4) * Int.from_nat(9) = -Int.from_nat(36)
} by {
    mul_neg_from_nat_left(4, 9)
    lib(nat).nat_mul_4_9
}

theorem int_mul_neg_left_5_1 {
    -Int.from_nat(5) * Int.from_nat(1) = -Int.from_nat(5)
} by {
    mul_neg_from_nat_left(5, 1)
    lib(nat).nat_mul_5_1
}

theorem int_mul_neg_left_5_2 {
    -Int.from_nat(5) * Int.from_nat(2) = -Int.from_nat(10)
} by {
    mul_neg_from_nat_left(5, 2)
    lib(nat).nat_mul_5_2
}

theorem int_mul_neg_left_5_3 {
    -Int.from_nat(5) * Int.from_nat(3) = -Int.from_nat(15)
} by {
    mul_neg_from_nat_left(5, 3)
    lib(nat).nat_mul_5_3
}

theorem int_mul_neg_left_5_4 {
    -Int.from_nat(5) * Int.from_nat(4) = -Int.from_nat(20)
} by {
    mul_neg_from_nat_left(5, 4)
    lib(nat).nat_mul_5_4
}

theorem int_mul_neg_left_5_5 {
    -Int.from_nat(5) * Int.from_nat(5) = -Int.from_nat(25)
} by {
    mul_neg_from_nat_left(5, 5)
    lib(nat).nat_mul_5_5
}

theorem int_mul_neg_left_5_6 {
    -Int.from_nat(5) * Int.from_nat(6) = -Int.from_nat(30)
} by {
    mul_neg_from_nat_left(5, 6)
    lib(nat).nat_mul_5_6
}

theorem int_mul_neg_left_5_7 {
    -Int.from_nat(5) * Int.from_nat(7) = -Int.from_nat(35)
} by {
    mul_neg_from_nat_left(5, 7)
    lib(nat).nat_mul_5_7
}

theorem int_mul_neg_left_5_8 {
    -Int.from_nat(5) * Int.from_nat(8) = -Int.from_nat(40)
} by {
    mul_neg_from_nat_left(5, 8)
    lib(nat).nat_mul_5_8
}

theorem int_mul_neg_left_5_9 {
    -Int.from_nat(5) * Int.from_nat(9) = -Int.from_nat(45)
} by {
    mul_neg_from_nat_left(5, 9)
    lib(nat).nat_mul_5_9
}

theorem int_mul_neg_left_6_1 {
    -Int.from_nat(6) * Int.from_nat(1) = -Int.from_nat(6)
} by {
    mul_neg_from_nat_left(6, 1)
    lib(nat).nat_mul_6_1
}

theorem int_mul_neg_left_6_2 {
    -Int.from_nat(6) * Int.from_nat(2) = -Int.from_nat(12)
} by {
    mul_neg_from_nat_left(6, 2)
    lib(nat).nat_mul_6_2
}

theorem int_mul_neg_left_6_3 {
    -Int.from_nat(6) * Int.from_nat(3) = -Int.from_nat(18)
} by {
    mul_neg_from_nat_left(6, 3)
    lib(nat).nat_mul_6_3
}

theorem int_mul_neg_left_6_4 {
    -Int.from_nat(6) * Int.from_nat(4) = -Int.from_nat(24)
} by {
    mul_neg_from_nat_left(6, 4)
    lib(nat).nat_mul_6_4
}

theorem int_mul_neg_left_6_5 {
    -Int.from_nat(6) * Int.from_nat(5) = -Int.from_nat(30)
} by {
    mul_neg_from_nat_left(6, 5)
    lib(nat).nat_mul_6_5
}

theorem int_mul_neg_left_6_6 {
    -Int.from_nat(6) * Int.from_nat(6) = -Int.from_nat(36)
} by {
    mul_neg_from_nat_left(6, 6)
    lib(nat).nat_mul_6_6
}

theorem int_mul_neg_left_6_7 {
    -Int.from_nat(6) * Int.from_nat(7) = -Int.from_nat(42)
} by {
    mul_neg_from_nat_left(6, 7)
    lib(nat).nat_mul_6_7
}

theorem int_mul_neg_left_6_8 {
    -Int.from_nat(6) * Int.from_nat(8) = -Int.from_nat(48)
} by {
    mul_neg_from_nat_left(6, 8)
    lib(nat).nat_mul_6_8
}

theorem int_mul_neg_left_6_9 {
    -Int.from_nat(6) * Int.from_nat(9) = -Int.from_nat(54)
} by {
    mul_neg_from_nat_left(6, 9)
    lib(nat).nat_mul_6_9
}

theorem int_mul_neg_left_7_1 {
    -Int.from_nat(7) * Int.from_nat(1) = -Int.from_nat(7)
} by {
    mul_neg_from_nat_left(7, 1)
    lib(nat).nat_mul_7_1
}

theorem int_mul_neg_left_7_2 {
    -Int.from_nat(7) * Int.from_nat(2) = -Int.from_nat(14)
} by {
    mul_neg_from_nat_left(7, 2)
    lib(nat).nat_mul_7_2
}

theorem int_mul_neg_left_7_3 {
    -Int.from_nat(7) * Int.from_nat(3) = -Int.from_nat(21)
} by {
    mul_neg_from_nat_left(7, 3)
    lib(nat).nat_mul_7_3
}

theorem int_mul_neg_left_7_4 {
    -Int.from_nat(7) * Int.from_nat(4) = -Int.from_nat(28)
} by {
    mul_neg_from_nat_left(7, 4)
    lib(nat).nat_mul_7_4
}

theorem int_mul_neg_left_7_5 {
    -Int.from_nat(7) * Int.from_nat(5) = -Int.from_nat(35)
} by {
    mul_neg_from_nat_left(7, 5)
    lib(nat).nat_mul_7_5
}

theorem int_mul_neg_left_7_6 {
    -Int.from_nat(7) * Int.from_nat(6) = -Int.from_nat(42)
} by {
    mul_neg_from_nat_left(7, 6)
    lib(nat).nat_mul_7_6
}

theorem int_mul_neg_left_7_7 {
    -Int.from_nat(7) * Int.from_nat(7) = -Int.from_nat(49)
} by {
    mul_neg_from_nat_left(7, 7)
    lib(nat).nat_mul_7_7
}

theorem int_mul_neg_left_7_8 {
    -Int.from_nat(7) * Int.from_nat(8) = -Int.from_nat(56)
} by {
    mul_neg_from_nat_left(7, 8)
    lib(nat).nat_mul_7_8
}

theorem int_mul_neg_left_7_9 {
    -Int.from_nat(7) * Int.from_nat(9) = -Int.from_nat(63)
} by {
    mul_neg_from_nat_left(7, 9)
    lib(nat).nat_mul_7_9
}

theorem int_mul_neg_left_8_1 {
    -Int.from_nat(8) * Int.from_nat(1) = -Int.from_nat(8)
} by {
    mul_neg_from_nat_left(8, 1)
    lib(nat).nat_mul_8_1
}

theorem int_mul_neg_left_8_2 {
    -Int.from_nat(8) * Int.from_nat(2) = -Int.from_nat(16)
} by {
    mul_neg_from_nat_left(8, 2)
    lib(nat).nat_mul_8_2
}

theorem int_mul_neg_left_8_3 {
    -Int.from_nat(8) * Int.from_nat(3) = -Int.from_nat(24)
} by {
    mul_neg_from_nat_left(8, 3)
    lib(nat).nat_mul_8_3
}

theorem int_mul_neg_left_8_4 {
    -Int.from_nat(8) * Int.from_nat(4) = -Int.from_nat(32)
} by {
    mul_neg_from_nat_left(8, 4)
    lib(nat).nat_mul_8_4
}

theorem int_mul_neg_left_8_5 {
    -Int.from_nat(8) * Int.from_nat(5) = -Int.from_nat(40)
} by {
    mul_neg_from_nat_left(8, 5)
    lib(nat).nat_mul_8_5
}

theorem int_mul_neg_left_8_6 {
    -Int.from_nat(8) * Int.from_nat(6) = -Int.from_nat(48)
} by {
    mul_neg_from_nat_left(8, 6)
    lib(nat).nat_mul_8_6
}

theorem int_mul_neg_left_8_7 {
    -Int.from_nat(8) * Int.from_nat(7) = -Int.from_nat(56)
} by {
    mul_neg_from_nat_left(8, 7)
    lib(nat).nat_mul_8_7
}

theorem int_mul_neg_left_8_8 {
    -Int.from_nat(8) * Int.from_nat(8) = -Int.from_nat(64)
} by {
    mul_neg_from_nat_left(8, 8)
    lib(nat).nat_mul_8_8
}

theorem int_mul_neg_left_8_9 {
    -Int.from_nat(8) * Int.from_nat(9) = -Int.from_nat(72)
} by {
    mul_neg_from_nat_left(8, 9)
    lib(nat).nat_mul_8_9
}

theorem int_mul_neg_left_9_1 {
    -Int.from_nat(9) * Int.from_nat(1) = -Int.from_nat(9)
} by {
    mul_neg_from_nat_left(9, 1)
    lib(nat).nat_mul_9_1
}

theorem int_mul_neg_left_9_2 {
    -Int.from_nat(9) * Int.from_nat(2) = -Int.from_nat(18)
} by {
    mul_neg_from_nat_left(9, 2)
    lib(nat).nat_mul_9_2
}

theorem int_mul_neg_left_9_3 {
    -Int.from_nat(9) * Int.from_nat(3) = -Int.from_nat(27)
} by {
    mul_neg_from_nat_left(9, 3)
    lib(nat).nat_mul_9_3
}

theorem int_mul_neg_left_9_4 {
    -Int.from_nat(9) * Int.from_nat(4) = -Int.from_nat(36)
} by {
    mul_neg_from_nat_left(9, 4)
    lib(nat).nat_mul_9_4
}

theorem int_mul_neg_left_9_5 {
    -Int.from_nat(9) * Int.from_nat(5) = -Int.from_nat(45)
} by {
    mul_neg_from_nat_left(9, 5)
    lib(nat).nat_mul_9_5
}

theorem int_mul_neg_left_9_6 {
    -Int.from_nat(9) * Int.from_nat(6) = -Int.from_nat(54)
} by {
    mul_neg_from_nat_left(9, 6)
    lib(nat).nat_mul_9_6
}

theorem int_mul_neg_left_9_7 {
    -Int.from_nat(9) * Int.from_nat(7) = -Int.from_nat(63)
} by {
    mul_neg_from_nat_left(9, 7)
    lib(nat).nat_mul_9_7
}

theorem int_mul_neg_left_9_8 {
    -Int.from_nat(9) * Int.from_nat(8) = -Int.from_nat(72)
} by {
    mul_neg_from_nat_left(9, 8)
    lib(nat).nat_mul_9_8
}

theorem int_mul_neg_left_9_9 {
    -Int.from_nat(9) * Int.from_nat(9) = -Int.from_nat(81)
} by {
    mul_neg_from_nat_left(9, 9)
    lib(nat).nat_mul_9_9
}

/// Negative-right integer multiplication facts for embedded natural numerals from 1 through 9.
theorem int_mul_neg_right_1_1 {
    Int.from_nat(1) * -Int.from_nat(1) = -Int.from_nat(1)
} by {
    mul_neg_from_nat_right(1, 1)
    lib(nat).nat_mul_1_1
}

theorem int_mul_neg_right_1_2 {
    Int.from_nat(1) * -Int.from_nat(2) = -Int.from_nat(2)
} by {
    mul_neg_from_nat_right(1, 2)
    lib(nat).nat_mul_1_2
}

theorem int_mul_neg_right_1_3 {
    Int.from_nat(1) * -Int.from_nat(3) = -Int.from_nat(3)
} by {
    mul_neg_from_nat_right(1, 3)
    lib(nat).nat_mul_1_3
}

theorem int_mul_neg_right_1_4 {
    Int.from_nat(1) * -Int.from_nat(4) = -Int.from_nat(4)
} by {
    mul_neg_from_nat_right(1, 4)
    lib(nat).nat_mul_1_4
}

theorem int_mul_neg_right_1_5 {
    Int.from_nat(1) * -Int.from_nat(5) = -Int.from_nat(5)
} by {
    mul_neg_from_nat_right(1, 5)
    lib(nat).nat_mul_1_5
}

theorem int_mul_neg_right_1_6 {
    Int.from_nat(1) * -Int.from_nat(6) = -Int.from_nat(6)
} by {
    mul_neg_from_nat_right(1, 6)
    lib(nat).nat_mul_1_6
}

theorem int_mul_neg_right_1_7 {
    Int.from_nat(1) * -Int.from_nat(7) = -Int.from_nat(7)
} by {
    mul_neg_from_nat_right(1, 7)
    lib(nat).nat_mul_1_7
}

theorem int_mul_neg_right_1_8 {
    Int.from_nat(1) * -Int.from_nat(8) = -Int.from_nat(8)
} by {
    mul_neg_from_nat_right(1, 8)
    lib(nat).nat_mul_1_8
}

theorem int_mul_neg_right_1_9 {
    Int.from_nat(1) * -Int.from_nat(9) = -Int.from_nat(9)
} by {
    mul_neg_from_nat_right(1, 9)
    lib(nat).nat_mul_1_9
}

theorem int_mul_neg_right_2_1 {
    Int.from_nat(2) * -Int.from_nat(1) = -Int.from_nat(2)
} by {
    mul_neg_from_nat_right(2, 1)
    lib(nat).nat_mul_2_1
}

theorem int_mul_neg_right_2_2 {
    Int.from_nat(2) * -Int.from_nat(2) = -Int.from_nat(4)
} by {
    mul_neg_from_nat_right(2, 2)
    lib(nat).nat_mul_2_2
}

theorem int_mul_neg_right_2_3 {
    Int.from_nat(2) * -Int.from_nat(3) = -Int.from_nat(6)
} by {
    mul_neg_from_nat_right(2, 3)
    lib(nat).nat_mul_2_3
}

theorem int_mul_neg_right_2_4 {
    Int.from_nat(2) * -Int.from_nat(4) = -Int.from_nat(8)
} by {
    mul_neg_from_nat_right(2, 4)
    lib(nat).nat_mul_2_4
}

theorem int_mul_neg_right_2_5 {
    Int.from_nat(2) * -Int.from_nat(5) = -Int.from_nat(10)
} by {
    mul_neg_from_nat_right(2, 5)
    lib(nat).nat_mul_2_5
}

theorem int_mul_neg_right_2_6 {
    Int.from_nat(2) * -Int.from_nat(6) = -Int.from_nat(12)
} by {
    mul_neg_from_nat_right(2, 6)
    lib(nat).nat_mul_2_6
}

theorem int_mul_neg_right_2_7 {
    Int.from_nat(2) * -Int.from_nat(7) = -Int.from_nat(14)
} by {
    mul_neg_from_nat_right(2, 7)
    lib(nat).nat_mul_2_7
}

theorem int_mul_neg_right_2_8 {
    Int.from_nat(2) * -Int.from_nat(8) = -Int.from_nat(16)
} by {
    mul_neg_from_nat_right(2, 8)
    lib(nat).nat_mul_2_8
}

theorem int_mul_neg_right_2_9 {
    Int.from_nat(2) * -Int.from_nat(9) = -Int.from_nat(18)
} by {
    mul_neg_from_nat_right(2, 9)
    lib(nat).nat_mul_2_9
}

theorem int_mul_neg_right_3_1 {
    Int.from_nat(3) * -Int.from_nat(1) = -Int.from_nat(3)
} by {
    mul_neg_from_nat_right(3, 1)
    lib(nat).nat_mul_3_1
}

theorem int_mul_neg_right_3_2 {
    Int.from_nat(3) * -Int.from_nat(2) = -Int.from_nat(6)
} by {
    mul_neg_from_nat_right(3, 2)
    lib(nat).nat_mul_3_2
}

theorem int_mul_neg_right_3_3 {
    Int.from_nat(3) * -Int.from_nat(3) = -Int.from_nat(9)
} by {
    mul_neg_from_nat_right(3, 3)
    lib(nat).nat_mul_3_3
}

theorem int_mul_neg_right_3_4 {
    Int.from_nat(3) * -Int.from_nat(4) = -Int.from_nat(12)
} by {
    mul_neg_from_nat_right(3, 4)
    lib(nat).nat_mul_3_4
}

theorem int_mul_neg_right_3_5 {
    Int.from_nat(3) * -Int.from_nat(5) = -Int.from_nat(15)
} by {
    mul_neg_from_nat_right(3, 5)
    lib(nat).nat_mul_3_5
}

theorem int_mul_neg_right_3_6 {
    Int.from_nat(3) * -Int.from_nat(6) = -Int.from_nat(18)
} by {
    mul_neg_from_nat_right(3, 6)
    lib(nat).nat_mul_3_6
}

theorem int_mul_neg_right_3_7 {
    Int.from_nat(3) * -Int.from_nat(7) = -Int.from_nat(21)
} by {
    mul_neg_from_nat_right(3, 7)
    lib(nat).nat_mul_3_7
}

theorem int_mul_neg_right_3_8 {
    Int.from_nat(3) * -Int.from_nat(8) = -Int.from_nat(24)
} by {
    mul_neg_from_nat_right(3, 8)
    lib(nat).nat_mul_3_8
}

theorem int_mul_neg_right_3_9 {
    Int.from_nat(3) * -Int.from_nat(9) = -Int.from_nat(27)
} by {
    mul_neg_from_nat_right(3, 9)
    lib(nat).nat_mul_3_9
}

theorem int_mul_neg_right_4_1 {
    Int.from_nat(4) * -Int.from_nat(1) = -Int.from_nat(4)
} by {
    mul_neg_from_nat_right(4, 1)
    lib(nat).nat_mul_4_1
}

theorem int_mul_neg_right_4_2 {
    Int.from_nat(4) * -Int.from_nat(2) = -Int.from_nat(8)
} by {
    mul_neg_from_nat_right(4, 2)
    lib(nat).nat_mul_4_2
}

theorem int_mul_neg_right_4_3 {
    Int.from_nat(4) * -Int.from_nat(3) = -Int.from_nat(12)
} by {
    mul_neg_from_nat_right(4, 3)
    lib(nat).nat_mul_4_3
}

theorem int_mul_neg_right_4_4 {
    Int.from_nat(4) * -Int.from_nat(4) = -Int.from_nat(16)
} by {
    mul_neg_from_nat_right(4, 4)
    lib(nat).nat_mul_4_4
}

theorem int_mul_neg_right_4_5 {
    Int.from_nat(4) * -Int.from_nat(5) = -Int.from_nat(20)
} by {
    mul_neg_from_nat_right(4, 5)
    lib(nat).nat_mul_4_5
}

theorem int_mul_neg_right_4_6 {
    Int.from_nat(4) * -Int.from_nat(6) = -Int.from_nat(24)
} by {
    mul_neg_from_nat_right(4, 6)
    lib(nat).nat_mul_4_6
}

theorem int_mul_neg_right_4_7 {
    Int.from_nat(4) * -Int.from_nat(7) = -Int.from_nat(28)
} by {
    mul_neg_from_nat_right(4, 7)
    lib(nat).nat_mul_4_7
}

theorem int_mul_neg_right_4_8 {
    Int.from_nat(4) * -Int.from_nat(8) = -Int.from_nat(32)
} by {
    mul_neg_from_nat_right(4, 8)
    lib(nat).nat_mul_4_8
}

theorem int_mul_neg_right_4_9 {
    Int.from_nat(4) * -Int.from_nat(9) = -Int.from_nat(36)
} by {
    mul_neg_from_nat_right(4, 9)
    lib(nat).nat_mul_4_9
}

theorem int_mul_neg_right_5_1 {
    Int.from_nat(5) * -Int.from_nat(1) = -Int.from_nat(5)
} by {
    mul_neg_from_nat_right(5, 1)
    lib(nat).nat_mul_5_1
}

theorem int_mul_neg_right_5_2 {
    Int.from_nat(5) * -Int.from_nat(2) = -Int.from_nat(10)
} by {
    mul_neg_from_nat_right(5, 2)
    lib(nat).nat_mul_5_2
}

theorem int_mul_neg_right_5_3 {
    Int.from_nat(5) * -Int.from_nat(3) = -Int.from_nat(15)
} by {
    mul_neg_from_nat_right(5, 3)
    lib(nat).nat_mul_5_3
}

theorem int_mul_neg_right_5_4 {
    Int.from_nat(5) * -Int.from_nat(4) = -Int.from_nat(20)
} by {
    mul_neg_from_nat_right(5, 4)
    lib(nat).nat_mul_5_4
}

theorem int_mul_neg_right_5_5 {
    Int.from_nat(5) * -Int.from_nat(5) = -Int.from_nat(25)
} by {
    mul_neg_from_nat_right(5, 5)
    lib(nat).nat_mul_5_5
}

theorem int_mul_neg_right_5_6 {
    Int.from_nat(5) * -Int.from_nat(6) = -Int.from_nat(30)
} by {
    mul_neg_from_nat_right(5, 6)
    lib(nat).nat_mul_5_6
}

theorem int_mul_neg_right_5_7 {
    Int.from_nat(5) * -Int.from_nat(7) = -Int.from_nat(35)
} by {
    mul_neg_from_nat_right(5, 7)
    lib(nat).nat_mul_5_7
}

theorem int_mul_neg_right_5_8 {
    Int.from_nat(5) * -Int.from_nat(8) = -Int.from_nat(40)
} by {
    mul_neg_from_nat_right(5, 8)
    lib(nat).nat_mul_5_8
}

theorem int_mul_neg_right_5_9 {
    Int.from_nat(5) * -Int.from_nat(9) = -Int.from_nat(45)
} by {
    mul_neg_from_nat_right(5, 9)
    lib(nat).nat_mul_5_9
}

theorem int_mul_neg_right_6_1 {
    Int.from_nat(6) * -Int.from_nat(1) = -Int.from_nat(6)
} by {
    mul_neg_from_nat_right(6, 1)
    lib(nat).nat_mul_6_1
}

theorem int_mul_neg_right_6_2 {
    Int.from_nat(6) * -Int.from_nat(2) = -Int.from_nat(12)
} by {
    mul_neg_from_nat_right(6, 2)
    lib(nat).nat_mul_6_2
}

theorem int_mul_neg_right_6_3 {
    Int.from_nat(6) * -Int.from_nat(3) = -Int.from_nat(18)
} by {
    mul_neg_from_nat_right(6, 3)
    lib(nat).nat_mul_6_3
}

theorem int_mul_neg_right_6_4 {
    Int.from_nat(6) * -Int.from_nat(4) = -Int.from_nat(24)
} by {
    mul_neg_from_nat_right(6, 4)
    lib(nat).nat_mul_6_4
}

theorem int_mul_neg_right_6_5 {
    Int.from_nat(6) * -Int.from_nat(5) = -Int.from_nat(30)
} by {
    mul_neg_from_nat_right(6, 5)
    lib(nat).nat_mul_6_5
}

theorem int_mul_neg_right_6_6 {
    Int.from_nat(6) * -Int.from_nat(6) = -Int.from_nat(36)
} by {
    mul_neg_from_nat_right(6, 6)
    lib(nat).nat_mul_6_6
}

theorem int_mul_neg_right_6_7 {
    Int.from_nat(6) * -Int.from_nat(7) = -Int.from_nat(42)
} by {
    mul_neg_from_nat_right(6, 7)
    lib(nat).nat_mul_6_7
}

theorem int_mul_neg_right_6_8 {
    Int.from_nat(6) * -Int.from_nat(8) = -Int.from_nat(48)
} by {
    mul_neg_from_nat_right(6, 8)
    lib(nat).nat_mul_6_8
}

theorem int_mul_neg_right_6_9 {
    Int.from_nat(6) * -Int.from_nat(9) = -Int.from_nat(54)
} by {
    mul_neg_from_nat_right(6, 9)
    lib(nat).nat_mul_6_9
}

theorem int_mul_neg_right_7_1 {
    Int.from_nat(7) * -Int.from_nat(1) = -Int.from_nat(7)
} by {
    mul_neg_from_nat_right(7, 1)
    lib(nat).nat_mul_7_1
}

theorem int_mul_neg_right_7_2 {
    Int.from_nat(7) * -Int.from_nat(2) = -Int.from_nat(14)
} by {
    mul_neg_from_nat_right(7, 2)
    lib(nat).nat_mul_7_2
}

theorem int_mul_neg_right_7_3 {
    Int.from_nat(7) * -Int.from_nat(3) = -Int.from_nat(21)
} by {
    mul_neg_from_nat_right(7, 3)
    lib(nat).nat_mul_7_3
}

theorem int_mul_neg_right_7_4 {
    Int.from_nat(7) * -Int.from_nat(4) = -Int.from_nat(28)
} by {
    mul_neg_from_nat_right(7, 4)
    lib(nat).nat_mul_7_4
}

theorem int_mul_neg_right_7_5 {
    Int.from_nat(7) * -Int.from_nat(5) = -Int.from_nat(35)
} by {
    mul_neg_from_nat_right(7, 5)
    lib(nat).nat_mul_7_5
}

theorem int_mul_neg_right_7_6 {
    Int.from_nat(7) * -Int.from_nat(6) = -Int.from_nat(42)
} by {
    mul_neg_from_nat_right(7, 6)
    lib(nat).nat_mul_7_6
}

theorem int_mul_neg_right_7_7 {
    Int.from_nat(7) * -Int.from_nat(7) = -Int.from_nat(49)
} by {
    mul_neg_from_nat_right(7, 7)
    lib(nat).nat_mul_7_7
}

theorem int_mul_neg_right_7_8 {
    Int.from_nat(7) * -Int.from_nat(8) = -Int.from_nat(56)
} by {
    mul_neg_from_nat_right(7, 8)
    lib(nat).nat_mul_7_8
}

theorem int_mul_neg_right_7_9 {
    Int.from_nat(7) * -Int.from_nat(9) = -Int.from_nat(63)
} by {
    mul_neg_from_nat_right(7, 9)
    lib(nat).nat_mul_7_9
}

theorem int_mul_neg_right_8_1 {
    Int.from_nat(8) * -Int.from_nat(1) = -Int.from_nat(8)
} by {
    mul_neg_from_nat_right(8, 1)
    lib(nat).nat_mul_8_1
}

theorem int_mul_neg_right_8_2 {
    Int.from_nat(8) * -Int.from_nat(2) = -Int.from_nat(16)
} by {
    mul_neg_from_nat_right(8, 2)
    lib(nat).nat_mul_8_2
}

theorem int_mul_neg_right_8_3 {
    Int.from_nat(8) * -Int.from_nat(3) = -Int.from_nat(24)
} by {
    mul_neg_from_nat_right(8, 3)
    lib(nat).nat_mul_8_3
}

theorem int_mul_neg_right_8_4 {
    Int.from_nat(8) * -Int.from_nat(4) = -Int.from_nat(32)
} by {
    mul_neg_from_nat_right(8, 4)
    lib(nat).nat_mul_8_4
}

theorem int_mul_neg_right_8_5 {
    Int.from_nat(8) * -Int.from_nat(5) = -Int.from_nat(40)
} by {
    mul_neg_from_nat_right(8, 5)
    lib(nat).nat_mul_8_5
}

theorem int_mul_neg_right_8_6 {
    Int.from_nat(8) * -Int.from_nat(6) = -Int.from_nat(48)
} by {
    mul_neg_from_nat_right(8, 6)
    lib(nat).nat_mul_8_6
}

theorem int_mul_neg_right_8_7 {
    Int.from_nat(8) * -Int.from_nat(7) = -Int.from_nat(56)
} by {
    mul_neg_from_nat_right(8, 7)
    lib(nat).nat_mul_8_7
}

theorem int_mul_neg_right_8_8 {
    Int.from_nat(8) * -Int.from_nat(8) = -Int.from_nat(64)
} by {
    mul_neg_from_nat_right(8, 8)
    lib(nat).nat_mul_8_8
}

theorem int_mul_neg_right_8_9 {
    Int.from_nat(8) * -Int.from_nat(9) = -Int.from_nat(72)
} by {
    mul_neg_from_nat_right(8, 9)
    lib(nat).nat_mul_8_9
}

theorem int_mul_neg_right_9_1 {
    Int.from_nat(9) * -Int.from_nat(1) = -Int.from_nat(9)
} by {
    mul_neg_from_nat_right(9, 1)
    lib(nat).nat_mul_9_1
}

theorem int_mul_neg_right_9_2 {
    Int.from_nat(9) * -Int.from_nat(2) = -Int.from_nat(18)
} by {
    mul_neg_from_nat_right(9, 2)
    lib(nat).nat_mul_9_2
}

theorem int_mul_neg_right_9_3 {
    Int.from_nat(9) * -Int.from_nat(3) = -Int.from_nat(27)
} by {
    mul_neg_from_nat_right(9, 3)
    lib(nat).nat_mul_9_3
}

theorem int_mul_neg_right_9_4 {
    Int.from_nat(9) * -Int.from_nat(4) = -Int.from_nat(36)
} by {
    mul_neg_from_nat_right(9, 4)
    lib(nat).nat_mul_9_4
}

theorem int_mul_neg_right_9_5 {
    Int.from_nat(9) * -Int.from_nat(5) = -Int.from_nat(45)
} by {
    mul_neg_from_nat_right(9, 5)
    lib(nat).nat_mul_9_5
}

theorem int_mul_neg_right_9_6 {
    Int.from_nat(9) * -Int.from_nat(6) = -Int.from_nat(54)
} by {
    mul_neg_from_nat_right(9, 6)
    lib(nat).nat_mul_9_6
}

theorem int_mul_neg_right_9_7 {
    Int.from_nat(9) * -Int.from_nat(7) = -Int.from_nat(63)
} by {
    mul_neg_from_nat_right(9, 7)
    lib(nat).nat_mul_9_7
}

theorem int_mul_neg_right_9_8 {
    Int.from_nat(9) * -Int.from_nat(8) = -Int.from_nat(72)
} by {
    mul_neg_from_nat_right(9, 8)
    lib(nat).nat_mul_9_8
}

theorem int_mul_neg_right_9_9 {
    Int.from_nat(9) * -Int.from_nat(9) = -Int.from_nat(81)
} by {
    mul_neg_from_nat_right(9, 9)
    lib(nat).nat_mul_9_9
}

/// Negative-negative integer multiplication facts for embedded natural numerals from 1 through 9.
theorem int_mul_neg_neg_1_1 {
    -Int.from_nat(1) * -Int.from_nat(1) = Int.from_nat(1)
} by {
    mul_neg_from_nat_neg(1, 1)
    lib(nat).nat_mul_1_1
}

theorem int_mul_neg_neg_1_2 {
    -Int.from_nat(1) * -Int.from_nat(2) = Int.from_nat(2)
} by {
    mul_neg_from_nat_neg(1, 2)
    lib(nat).nat_mul_1_2
}

theorem int_mul_neg_neg_1_3 {
    -Int.from_nat(1) * -Int.from_nat(3) = Int.from_nat(3)
} by {
    mul_neg_from_nat_neg(1, 3)
    lib(nat).nat_mul_1_3
}

theorem int_mul_neg_neg_1_4 {
    -Int.from_nat(1) * -Int.from_nat(4) = Int.from_nat(4)
} by {
    mul_neg_from_nat_neg(1, 4)
    lib(nat).nat_mul_1_4
}

theorem int_mul_neg_neg_1_5 {
    -Int.from_nat(1) * -Int.from_nat(5) = Int.from_nat(5)
} by {
    mul_neg_from_nat_neg(1, 5)
    lib(nat).nat_mul_1_5
}

theorem int_mul_neg_neg_1_6 {
    -Int.from_nat(1) * -Int.from_nat(6) = Int.from_nat(6)
} by {
    mul_neg_from_nat_neg(1, 6)
    lib(nat).nat_mul_1_6
}

theorem int_mul_neg_neg_1_7 {
    -Int.from_nat(1) * -Int.from_nat(7) = Int.from_nat(7)
} by {
    mul_neg_from_nat_neg(1, 7)
    lib(nat).nat_mul_1_7
}

theorem int_mul_neg_neg_1_8 {
    -Int.from_nat(1) * -Int.from_nat(8) = Int.from_nat(8)
} by {
    mul_neg_from_nat_neg(1, 8)
    lib(nat).nat_mul_1_8
}

theorem int_mul_neg_neg_1_9 {
    -Int.from_nat(1) * -Int.from_nat(9) = Int.from_nat(9)
} by {
    mul_neg_from_nat_neg(1, 9)
    lib(nat).nat_mul_1_9
}

theorem int_mul_neg_neg_2_1 {
    -Int.from_nat(2) * -Int.from_nat(1) = Int.from_nat(2)
} by {
    mul_neg_from_nat_neg(2, 1)
    lib(nat).nat_mul_2_1
}

theorem int_mul_neg_neg_2_2 {
    -Int.from_nat(2) * -Int.from_nat(2) = Int.from_nat(4)
} by {
    mul_neg_from_nat_neg(2, 2)
    lib(nat).nat_mul_2_2
}

theorem int_mul_neg_neg_2_3 {
    -Int.from_nat(2) * -Int.from_nat(3) = Int.from_nat(6)
} by {
    mul_neg_from_nat_neg(2, 3)
    lib(nat).nat_mul_2_3
}

theorem int_mul_neg_neg_2_4 {
    -Int.from_nat(2) * -Int.from_nat(4) = Int.from_nat(8)
} by {
    mul_neg_from_nat_neg(2, 4)
    lib(nat).nat_mul_2_4
}

theorem int_mul_neg_neg_2_5 {
    -Int.from_nat(2) * -Int.from_nat(5) = Int.from_nat(10)
} by {
    mul_neg_from_nat_neg(2, 5)
    lib(nat).nat_mul_2_5
}

theorem int_mul_neg_neg_2_6 {
    -Int.from_nat(2) * -Int.from_nat(6) = Int.from_nat(12)
} by {
    mul_neg_from_nat_neg(2, 6)
    lib(nat).nat_mul_2_6
}

theorem int_mul_neg_neg_2_7 {
    -Int.from_nat(2) * -Int.from_nat(7) = Int.from_nat(14)
} by {
    mul_neg_from_nat_neg(2, 7)
    lib(nat).nat_mul_2_7
}

theorem int_mul_neg_neg_2_8 {
    -Int.from_nat(2) * -Int.from_nat(8) = Int.from_nat(16)
} by {
    mul_neg_from_nat_neg(2, 8)
    lib(nat).nat_mul_2_8
}

theorem int_mul_neg_neg_2_9 {
    -Int.from_nat(2) * -Int.from_nat(9) = Int.from_nat(18)
} by {
    mul_neg_from_nat_neg(2, 9)
    lib(nat).nat_mul_2_9
}

theorem int_mul_neg_neg_3_1 {
    -Int.from_nat(3) * -Int.from_nat(1) = Int.from_nat(3)
} by {
    mul_neg_from_nat_neg(3, 1)
    lib(nat).nat_mul_3_1
}

theorem int_mul_neg_neg_3_2 {
    -Int.from_nat(3) * -Int.from_nat(2) = Int.from_nat(6)
} by {
    mul_neg_from_nat_neg(3, 2)
    lib(nat).nat_mul_3_2
}

theorem int_mul_neg_neg_3_3 {
    -Int.from_nat(3) * -Int.from_nat(3) = Int.from_nat(9)
} by {
    mul_neg_from_nat_neg(3, 3)
    lib(nat).nat_mul_3_3
}

theorem int_mul_neg_neg_3_4 {
    -Int.from_nat(3) * -Int.from_nat(4) = Int.from_nat(12)
} by {
    mul_neg_from_nat_neg(3, 4)
    lib(nat).nat_mul_3_4
}

theorem int_mul_neg_neg_3_5 {
    -Int.from_nat(3) * -Int.from_nat(5) = Int.from_nat(15)
} by {
    mul_neg_from_nat_neg(3, 5)
    lib(nat).nat_mul_3_5
}

theorem int_mul_neg_neg_3_6 {
    -Int.from_nat(3) * -Int.from_nat(6) = Int.from_nat(18)
} by {
    mul_neg_from_nat_neg(3, 6)
    lib(nat).nat_mul_3_6
}

theorem int_mul_neg_neg_3_7 {
    -Int.from_nat(3) * -Int.from_nat(7) = Int.from_nat(21)
} by {
    mul_neg_from_nat_neg(3, 7)
    lib(nat).nat_mul_3_7
}

theorem int_mul_neg_neg_3_8 {
    -Int.from_nat(3) * -Int.from_nat(8) = Int.from_nat(24)
} by {
    mul_neg_from_nat_neg(3, 8)
    lib(nat).nat_mul_3_8
}

theorem int_mul_neg_neg_3_9 {
    -Int.from_nat(3) * -Int.from_nat(9) = Int.from_nat(27)
} by {
    mul_neg_from_nat_neg(3, 9)
    lib(nat).nat_mul_3_9
}

theorem int_mul_neg_neg_4_1 {
    -Int.from_nat(4) * -Int.from_nat(1) = Int.from_nat(4)
} by {
    mul_neg_from_nat_neg(4, 1)
    lib(nat).nat_mul_4_1
}

theorem int_mul_neg_neg_4_2 {
    -Int.from_nat(4) * -Int.from_nat(2) = Int.from_nat(8)
} by {
    mul_neg_from_nat_neg(4, 2)
    lib(nat).nat_mul_4_2
}

theorem int_mul_neg_neg_4_3 {
    -Int.from_nat(4) * -Int.from_nat(3) = Int.from_nat(12)
} by {
    mul_neg_from_nat_neg(4, 3)
    lib(nat).nat_mul_4_3
}

theorem int_mul_neg_neg_4_4 {
    -Int.from_nat(4) * -Int.from_nat(4) = Int.from_nat(16)
} by {
    mul_neg_from_nat_neg(4, 4)
    lib(nat).nat_mul_4_4
}

theorem int_mul_neg_neg_4_5 {
    -Int.from_nat(4) * -Int.from_nat(5) = Int.from_nat(20)
} by {
    mul_neg_from_nat_neg(4, 5)
    lib(nat).nat_mul_4_5
}

theorem int_mul_neg_neg_4_6 {
    -Int.from_nat(4) * -Int.from_nat(6) = Int.from_nat(24)
} by {
    mul_neg_from_nat_neg(4, 6)
    lib(nat).nat_mul_4_6
}

theorem int_mul_neg_neg_4_7 {
    -Int.from_nat(4) * -Int.from_nat(7) = Int.from_nat(28)
} by {
    mul_neg_from_nat_neg(4, 7)
    lib(nat).nat_mul_4_7
}

theorem int_mul_neg_neg_4_8 {
    -Int.from_nat(4) * -Int.from_nat(8) = Int.from_nat(32)
} by {
    mul_neg_from_nat_neg(4, 8)
    lib(nat).nat_mul_4_8
}

theorem int_mul_neg_neg_4_9 {
    -Int.from_nat(4) * -Int.from_nat(9) = Int.from_nat(36)
} by {
    mul_neg_from_nat_neg(4, 9)
    lib(nat).nat_mul_4_9
}

theorem int_mul_neg_neg_5_1 {
    -Int.from_nat(5) * -Int.from_nat(1) = Int.from_nat(5)
} by {
    mul_neg_from_nat_neg(5, 1)
    lib(nat).nat_mul_5_1
}

theorem int_mul_neg_neg_5_2 {
    -Int.from_nat(5) * -Int.from_nat(2) = Int.from_nat(10)
} by {
    mul_neg_from_nat_neg(5, 2)
    lib(nat).nat_mul_5_2
}

theorem int_mul_neg_neg_5_3 {
    -Int.from_nat(5) * -Int.from_nat(3) = Int.from_nat(15)
} by {
    mul_neg_from_nat_neg(5, 3)
    lib(nat).nat_mul_5_3
}

theorem int_mul_neg_neg_5_4 {
    -Int.from_nat(5) * -Int.from_nat(4) = Int.from_nat(20)
} by {
    mul_neg_from_nat_neg(5, 4)
    lib(nat).nat_mul_5_4
}

theorem int_mul_neg_neg_5_5 {
    -Int.from_nat(5) * -Int.from_nat(5) = Int.from_nat(25)
} by {
    mul_neg_from_nat_neg(5, 5)
    lib(nat).nat_mul_5_5
}

theorem int_mul_neg_neg_5_6 {
    -Int.from_nat(5) * -Int.from_nat(6) = Int.from_nat(30)
} by {
    mul_neg_from_nat_neg(5, 6)
    lib(nat).nat_mul_5_6
}

theorem int_mul_neg_neg_5_7 {
    -Int.from_nat(5) * -Int.from_nat(7) = Int.from_nat(35)
} by {
    mul_neg_from_nat_neg(5, 7)
    lib(nat).nat_mul_5_7
}

theorem int_mul_neg_neg_5_8 {
    -Int.from_nat(5) * -Int.from_nat(8) = Int.from_nat(40)
} by {
    mul_neg_from_nat_neg(5, 8)
    lib(nat).nat_mul_5_8
}

theorem int_mul_neg_neg_5_9 {
    -Int.from_nat(5) * -Int.from_nat(9) = Int.from_nat(45)
} by {
    mul_neg_from_nat_neg(5, 9)
    lib(nat).nat_mul_5_9
}

theorem int_mul_neg_neg_6_1 {
    -Int.from_nat(6) * -Int.from_nat(1) = Int.from_nat(6)
} by {
    mul_neg_from_nat_neg(6, 1)
    lib(nat).nat_mul_6_1
}

theorem int_mul_neg_neg_6_2 {
    -Int.from_nat(6) * -Int.from_nat(2) = Int.from_nat(12)
} by {
    mul_neg_from_nat_neg(6, 2)
    lib(nat).nat_mul_6_2
}

theorem int_mul_neg_neg_6_3 {
    -Int.from_nat(6) * -Int.from_nat(3) = Int.from_nat(18)
} by {
    mul_neg_from_nat_neg(6, 3)
    lib(nat).nat_mul_6_3
}

theorem int_mul_neg_neg_6_4 {
    -Int.from_nat(6) * -Int.from_nat(4) = Int.from_nat(24)
} by {
    mul_neg_from_nat_neg(6, 4)
    lib(nat).nat_mul_6_4
}

theorem int_mul_neg_neg_6_5 {
    -Int.from_nat(6) * -Int.from_nat(5) = Int.from_nat(30)
} by {
    mul_neg_from_nat_neg(6, 5)
    lib(nat).nat_mul_6_5
}

theorem int_mul_neg_neg_6_6 {
    -Int.from_nat(6) * -Int.from_nat(6) = Int.from_nat(36)
} by {
    mul_neg_from_nat_neg(6, 6)
    lib(nat).nat_mul_6_6
}

theorem int_mul_neg_neg_6_7 {
    -Int.from_nat(6) * -Int.from_nat(7) = Int.from_nat(42)
} by {
    mul_neg_from_nat_neg(6, 7)
    lib(nat).nat_mul_6_7
}

theorem int_mul_neg_neg_6_8 {
    -Int.from_nat(6) * -Int.from_nat(8) = Int.from_nat(48)
} by {
    mul_neg_from_nat_neg(6, 8)
    lib(nat).nat_mul_6_8
}

theorem int_mul_neg_neg_6_9 {
    -Int.from_nat(6) * -Int.from_nat(9) = Int.from_nat(54)
} by {
    mul_neg_from_nat_neg(6, 9)
    lib(nat).nat_mul_6_9
}

theorem int_mul_neg_neg_7_1 {
    -Int.from_nat(7) * -Int.from_nat(1) = Int.from_nat(7)
} by {
    mul_neg_from_nat_neg(7, 1)
    lib(nat).nat_mul_7_1
}

theorem int_mul_neg_neg_7_2 {
    -Int.from_nat(7) * -Int.from_nat(2) = Int.from_nat(14)
} by {
    mul_neg_from_nat_neg(7, 2)
    lib(nat).nat_mul_7_2
}

theorem int_mul_neg_neg_7_3 {
    -Int.from_nat(7) * -Int.from_nat(3) = Int.from_nat(21)
} by {
    mul_neg_from_nat_neg(7, 3)
    lib(nat).nat_mul_7_3
}

theorem int_mul_neg_neg_7_4 {
    -Int.from_nat(7) * -Int.from_nat(4) = Int.from_nat(28)
} by {
    mul_neg_from_nat_neg(7, 4)
    lib(nat).nat_mul_7_4
}

theorem int_mul_neg_neg_7_5 {
    -Int.from_nat(7) * -Int.from_nat(5) = Int.from_nat(35)
} by {
    mul_neg_from_nat_neg(7, 5)
    lib(nat).nat_mul_7_5
}

theorem int_mul_neg_neg_7_6 {
    -Int.from_nat(7) * -Int.from_nat(6) = Int.from_nat(42)
} by {
    mul_neg_from_nat_neg(7, 6)
    lib(nat).nat_mul_7_6
}

theorem int_mul_neg_neg_7_7 {
    -Int.from_nat(7) * -Int.from_nat(7) = Int.from_nat(49)
} by {
    mul_neg_from_nat_neg(7, 7)
    lib(nat).nat_mul_7_7
}

theorem int_mul_neg_neg_7_8 {
    -Int.from_nat(7) * -Int.from_nat(8) = Int.from_nat(56)
} by {
    mul_neg_from_nat_neg(7, 8)
    lib(nat).nat_mul_7_8
}

theorem int_mul_neg_neg_7_9 {
    -Int.from_nat(7) * -Int.from_nat(9) = Int.from_nat(63)
} by {
    mul_neg_from_nat_neg(7, 9)
    lib(nat).nat_mul_7_9
}

theorem int_mul_neg_neg_8_1 {
    -Int.from_nat(8) * -Int.from_nat(1) = Int.from_nat(8)
} by {
    mul_neg_from_nat_neg(8, 1)
    lib(nat).nat_mul_8_1
}

theorem int_mul_neg_neg_8_2 {
    -Int.from_nat(8) * -Int.from_nat(2) = Int.from_nat(16)
} by {
    mul_neg_from_nat_neg(8, 2)
    lib(nat).nat_mul_8_2
}

theorem int_mul_neg_neg_8_3 {
    -Int.from_nat(8) * -Int.from_nat(3) = Int.from_nat(24)
} by {
    mul_neg_from_nat_neg(8, 3)
    lib(nat).nat_mul_8_3
}

theorem int_mul_neg_neg_8_4 {
    -Int.from_nat(8) * -Int.from_nat(4) = Int.from_nat(32)
} by {
    mul_neg_from_nat_neg(8, 4)
    lib(nat).nat_mul_8_4
}

theorem int_mul_neg_neg_8_5 {
    -Int.from_nat(8) * -Int.from_nat(5) = Int.from_nat(40)
} by {
    mul_neg_from_nat_neg(8, 5)
    lib(nat).nat_mul_8_5
}

theorem int_mul_neg_neg_8_6 {
    -Int.from_nat(8) * -Int.from_nat(6) = Int.from_nat(48)
} by {
    mul_neg_from_nat_neg(8, 6)
    lib(nat).nat_mul_8_6
}

theorem int_mul_neg_neg_8_7 {
    -Int.from_nat(8) * -Int.from_nat(7) = Int.from_nat(56)
} by {
    mul_neg_from_nat_neg(8, 7)
    lib(nat).nat_mul_8_7
}

theorem int_mul_neg_neg_8_8 {
    -Int.from_nat(8) * -Int.from_nat(8) = Int.from_nat(64)
} by {
    mul_neg_from_nat_neg(8, 8)
    lib(nat).nat_mul_8_8
}

theorem int_mul_neg_neg_8_9 {
    -Int.from_nat(8) * -Int.from_nat(9) = Int.from_nat(72)
} by {
    mul_neg_from_nat_neg(8, 9)
    lib(nat).nat_mul_8_9
}

theorem int_mul_neg_neg_9_1 {
    -Int.from_nat(9) * -Int.from_nat(1) = Int.from_nat(9)
} by {
    mul_neg_from_nat_neg(9, 1)
    lib(nat).nat_mul_9_1
}

theorem int_mul_neg_neg_9_2 {
    -Int.from_nat(9) * -Int.from_nat(2) = Int.from_nat(18)
} by {
    mul_neg_from_nat_neg(9, 2)
    lib(nat).nat_mul_9_2
}

theorem int_mul_neg_neg_9_3 {
    -Int.from_nat(9) * -Int.from_nat(3) = Int.from_nat(27)
} by {
    mul_neg_from_nat_neg(9, 3)
    lib(nat).nat_mul_9_3
}

theorem int_mul_neg_neg_9_4 {
    -Int.from_nat(9) * -Int.from_nat(4) = Int.from_nat(36)
} by {
    mul_neg_from_nat_neg(9, 4)
    lib(nat).nat_mul_9_4
}

theorem int_mul_neg_neg_9_5 {
    -Int.from_nat(9) * -Int.from_nat(5) = Int.from_nat(45)
} by {
    mul_neg_from_nat_neg(9, 5)
    lib(nat).nat_mul_9_5
}

theorem int_mul_neg_neg_9_6 {
    -Int.from_nat(9) * -Int.from_nat(6) = Int.from_nat(54)
} by {
    mul_neg_from_nat_neg(9, 6)
    lib(nat).nat_mul_9_6
}

theorem int_mul_neg_neg_9_7 {
    -Int.from_nat(9) * -Int.from_nat(7) = Int.from_nat(63)
} by {
    mul_neg_from_nat_neg(9, 7)
    lib(nat).nat_mul_9_7
}

theorem int_mul_neg_neg_9_8 {
    -Int.from_nat(9) * -Int.from_nat(8) = Int.from_nat(72)
} by {
    mul_neg_from_nat_neg(9, 8)
    lib(nat).nat_mul_9_8
}

theorem int_mul_neg_neg_9_9 {
    -Int.from_nat(9) * -Int.from_nat(9) = Int.from_nat(81)
} by {
    mul_neg_from_nat_neg(9, 9)
    lib(nat).nat_mul_9_9
}

/// Zero times zero.
theorem int_mul_zero_zero {
    Int.0 * Int.0 = Int.0
} by {
    mul_zero_left(Int.0)
}

/// Zero-row and zero-column facts for embedded natural numerals from 1 through 9.
theorem int_mul_zero_pos_1 {
    Int.0 * Int.from_nat(1) = Int.0
} by {
    mul_zero_left(Int.from_nat(1))
}

theorem int_mul_pos_zero_1 {
    Int.from_nat(1) * Int.0 = Int.0
} by {
    mul_zero_right(Int.from_nat(1))
}

theorem int_mul_zero_neg_1 {
    Int.0 * -Int.from_nat(1) = Int.0
} by {
    mul_zero_left(-Int.from_nat(1))
}

theorem int_mul_neg_zero_1 {
    -Int.from_nat(1) * Int.0 = Int.0
} by {
    mul_zero_right(-Int.from_nat(1))
}

theorem int_mul_zero_pos_2 {
    Int.0 * Int.from_nat(2) = Int.0
} by {
    mul_zero_left(Int.from_nat(2))
}

theorem int_mul_pos_zero_2 {
    Int.from_nat(2) * Int.0 = Int.0
} by {
    mul_zero_right(Int.from_nat(2))
}

theorem int_mul_zero_neg_2 {
    Int.0 * -Int.from_nat(2) = Int.0
} by {
    mul_zero_left(-Int.from_nat(2))
}

theorem int_mul_neg_zero_2 {
    -Int.from_nat(2) * Int.0 = Int.0
} by {
    mul_zero_right(-Int.from_nat(2))
}

theorem int_mul_zero_pos_3 {
    Int.0 * Int.from_nat(3) = Int.0
} by {
    mul_zero_left(Int.from_nat(3))
}

theorem int_mul_pos_zero_3 {
    Int.from_nat(3) * Int.0 = Int.0
} by {
    mul_zero_right(Int.from_nat(3))
}

theorem int_mul_zero_neg_3 {
    Int.0 * -Int.from_nat(3) = Int.0
} by {
    mul_zero_left(-Int.from_nat(3))
}

theorem int_mul_neg_zero_3 {
    -Int.from_nat(3) * Int.0 = Int.0
} by {
    mul_zero_right(-Int.from_nat(3))
}

theorem int_mul_zero_pos_4 {
    Int.0 * Int.from_nat(4) = Int.0
} by {
    mul_zero_left(Int.from_nat(4))
}

theorem int_mul_pos_zero_4 {
    Int.from_nat(4) * Int.0 = Int.0
} by {
    mul_zero_right(Int.from_nat(4))
}

theorem int_mul_zero_neg_4 {
    Int.0 * -Int.from_nat(4) = Int.0
} by {
    mul_zero_left(-Int.from_nat(4))
}

theorem int_mul_neg_zero_4 {
    -Int.from_nat(4) * Int.0 = Int.0
} by {
    mul_zero_right(-Int.from_nat(4))
}

theorem int_mul_zero_pos_5 {
    Int.0 * Int.from_nat(5) = Int.0
} by {
    mul_zero_left(Int.from_nat(5))
}

theorem int_mul_pos_zero_5 {
    Int.from_nat(5) * Int.0 = Int.0
} by {
    mul_zero_right(Int.from_nat(5))
}

theorem int_mul_zero_neg_5 {
    Int.0 * -Int.from_nat(5) = Int.0
} by {
    mul_zero_left(-Int.from_nat(5))
}

theorem int_mul_neg_zero_5 {
    -Int.from_nat(5) * Int.0 = Int.0
} by {
    mul_zero_right(-Int.from_nat(5))
}

theorem int_mul_zero_pos_6 {
    Int.0 * Int.from_nat(6) = Int.0
} by {
    mul_zero_left(Int.from_nat(6))
}

theorem int_mul_pos_zero_6 {
    Int.from_nat(6) * Int.0 = Int.0
} by {
    mul_zero_right(Int.from_nat(6))
}

theorem int_mul_zero_neg_6 {
    Int.0 * -Int.from_nat(6) = Int.0
} by {
    mul_zero_left(-Int.from_nat(6))
}

theorem int_mul_neg_zero_6 {
    -Int.from_nat(6) * Int.0 = Int.0
} by {
    mul_zero_right(-Int.from_nat(6))
}

theorem int_mul_zero_pos_7 {
    Int.0 * Int.from_nat(7) = Int.0
} by {
    mul_zero_left(Int.from_nat(7))
}

theorem int_mul_pos_zero_7 {
    Int.from_nat(7) * Int.0 = Int.0
} by {
    mul_zero_right(Int.from_nat(7))
}

theorem int_mul_zero_neg_7 {
    Int.0 * -Int.from_nat(7) = Int.0
} by {
    mul_zero_left(-Int.from_nat(7))
}

theorem int_mul_neg_zero_7 {
    -Int.from_nat(7) * Int.0 = Int.0
} by {
    mul_zero_right(-Int.from_nat(7))
}

theorem int_mul_zero_pos_8 {
    Int.0 * Int.from_nat(8) = Int.0
} by {
    mul_zero_left(Int.from_nat(8))
}

theorem int_mul_pos_zero_8 {
    Int.from_nat(8) * Int.0 = Int.0
} by {
    mul_zero_right(Int.from_nat(8))
}

theorem int_mul_zero_neg_8 {
    Int.0 * -Int.from_nat(8) = Int.0
} by {
    mul_zero_left(-Int.from_nat(8))
}

theorem int_mul_neg_zero_8 {
    -Int.from_nat(8) * Int.0 = Int.0
} by {
    mul_zero_right(-Int.from_nat(8))
}

theorem int_mul_zero_pos_9 {
    Int.0 * Int.from_nat(9) = Int.0
} by {
    mul_zero_left(Int.from_nat(9))
}

theorem int_mul_pos_zero_9 {
    Int.from_nat(9) * Int.0 = Int.0
} by {
    mul_zero_right(Int.from_nat(9))
}

theorem int_mul_zero_neg_9 {
    Int.0 * -Int.from_nat(9) = Int.0
} by {
    mul_zero_left(-Int.from_nat(9))
}

theorem int_mul_neg_zero_9 {
    -Int.from_nat(9) * Int.0 = Int.0
} by {
    mul_zero_right(-Int.from_nat(9))
}
