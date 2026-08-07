from int.lattice import Int, div_from_nat, div_trans, div_imp_div_abs, div_abs_imp_div
from int.int_base import abs, abs_mul, abs_from_nat, abs_neg
from int.int_gcd_extra import nat_gcd_self
from nat import Nat
from nat import gcd_mul_lcm, gcd_comm, gcd_nonzero_left, mul_cancel_left, cofactor,
    gcd_divides_left, gcd_divides_right, gcd_mult_right, divides_gcd
numerals Int

/// Local Nat helper: zero is absorbing for lcm on the left.
theorem nat_lcm_zero_left_for_int_lcm(a: Nat) {
    Nat.0.lcm(a) = Nat.0
} by {
    if a = Nat.0 {
        Nat.0.gcd(a) = Nat.0
    } else {
        Nat.0 * a = Nat.0
        a * Nat.0.lcm(a) = Nat.0
        Nat.0.lcm(a) = Nat.0
    }
}

/// Local Nat helper: zero is absorbing for lcm on the right.
theorem nat_lcm_zero_right_for_int_lcm(a: Nat) {
    a.lcm(Nat.0) = Nat.0
} by {
    if a = Nat.0 {
        a.gcd(Nat.0) = Nat.0
    } else {
        a * Nat.0 = Nat.0
        a * a.lcm(Nat.0) = Nat.0
        a.lcm(Nat.0) = Nat.0
    }
}

/// Local Nat helper: lcm is symmetric.
theorem nat_lcm_comm_for_int_lcm(a: Nat, b: Nat) {
    a.lcm(b) = b.lcm(a)
} by {
    gcd_comm(a, b)
    a.gcd(b) = b.gcd(a)
    gcd_mul_lcm(a, b)
    gcd_mul_lcm(b, a)
    a.gcd(b) * a.lcm(b) = a * b
    b.gcd(a) * b.lcm(a) = b * a
    b * a = a * b
    a.gcd(b) * b.lcm(a) = a * b
    a.gcd(b) * a.lcm(b) = a.gcd(b) * b.lcm(a)
    if a.gcd(b) = Nat.0 {
        gcd_nonzero_left(a, b)
        gcd_comm(a, b)
        gcd_nonzero_left(b, a)
        nat_lcm_zero_left_for_int_lcm(b)
        nat_lcm_zero_right_for_int_lcm(b)
        b.lcm(a) = Nat.0
    } else {
        a.lcm(b) = b.lcm(a)
    }
}

/// Local Nat helper: lcm of a natural with itself is itself.
theorem nat_lcm_self_for_int_lcm(a: Nat) {
    a.lcm(a) = a
} by {
    if a = Nat.0 {
        nat_lcm_zero_left_for_int_lcm(Nat.0)
        a.lcm(a) = a
    } else {
        nat_gcd_self(a)
        gcd_mul_lcm(a, a)
        mul_cancel_left(a, a.lcm(a), a)
        a.lcm(a) = a
    }
}

attributes Int {
    /// The least common multiple of this integer and b, defined as the
    /// nonneg integer with absolute value equal to the natural-number lcm
    /// of the two absolute values.
    define lcm(self, b: Int) -> Int {
        Int.from_nat(abs(self).lcm(abs(b)))
    }
}

/// The integer lcm is always nonnegative.
theorem int_lcm_nonneg(a: Int, b: Int) {
    not a.lcm(b).is_negative
}

/// Bridge: the integer lcm is the embedding of the natural lcm of the absolute values.
theorem int_lcm_from_nat_abs(a: Int, b: Int) {
    a.lcm(b) = Int.from_nat(abs(a).lcm(abs(b)))
}

/// Zero is absorbing for integer lcm on the left.
theorem int_lcm_zero_left(a: Int) {
    Int.0.lcm(a) = Int.0
} by {
    nat_lcm_zero_left_for_int_lcm(abs(a))
}

/// Zero is absorbing for integer lcm on the right.
theorem int_lcm_zero_right(a: Int) {
    a.lcm(Int.0) = Int.0
} by {
    nat_lcm_zero_right_for_int_lcm(abs(a))
}

/// Integer lcm is symmetric.
theorem int_lcm_comm(a: Int, b: Int) {
    a.lcm(b) = b.lcm(a)
} by {
    nat_lcm_comm_for_int_lcm(abs(a), abs(b))
}

/// Integer lcm is unchanged by negating the left input.
theorem int_lcm_neg_left(a: Int, b: Int) {
    (-a).lcm(b) = a.lcm(b)
} by {
    abs_neg(a)
}

/// Integer lcm is unchanged by negating the right input.
theorem int_lcm_neg_right(a: Int, b: Int) {
    a.lcm(-b) = a.lcm(b)
} by {
    abs_neg(b)
}

/// Integer lcm is unchanged by negating both inputs.
theorem int_lcm_neg_neg(a: Int, b: Int) {
    (-a).lcm(-b) = a.lcm(b)
} by {
    int_lcm_neg_left(a, -b)
    int_lcm_neg_right(a, b)
}

/// The lcm of an integer with itself is its nonnegative associate.
theorem int_lcm_self(a: Int) {
    a.lcm(a) = Int.from_nat(abs(a))
} by {
    nat_lcm_self_for_int_lcm(abs(a))
}

/// The natural absolute value of the integer lcm is the natural lcm of absolute values.
theorem int_abs_lcm(a: Int, b: Int) {
    abs(a.lcm(b)) = abs(a).lcm(abs(b))
} by {
    abs_from_nat(abs(a).lcm(abs(b)))
}

/// Defining product identity on the integers: gcd times lcm equals the
/// natural-number product of the absolute values, lifted to Int.
theorem int_gcd_mul_lcm(a: Int, b: Int) {
    a.gcd(b) * a.lcm(b) = Int.from_nat(abs(a) * abs(b))
} by {
    gcd_mul_lcm(abs(a), abs(b))
    Int.from_nat(abs(a).gcd(abs(b)) * abs(a).lcm(abs(b))) = Int.from_nat(abs(a) * abs(b))
}

/// Defining product identity in absolute-value form: gcd times lcm has the
/// same magnitude as the product a * b.
theorem int_gcd_mul_lcm_abs(a: Int, b: Int) {
    abs(a.gcd(b) * a.lcm(b)) = abs(a * b)
} by {
    int_gcd_mul_lcm(a, b)
    abs_mul(a, b)
}

/// Local Nat helper: the left input divides the lcm.
theorem nat_lcm_divides_left_for_int_lcm(a: Nat, b: Nat) {
    a.divides(a.lcm(b))
} by {
    if a = Nat.0 {
        nat_lcm_zero_left_for_int_lcm(b)
        a = Nat.0
        Nat.0.lcm(b) = Nat.0
        a.lcm(b) = Nat.0
        Nat.0 * Nat.0 = Nat.0
        a.divides(a.lcm(b))
    }
    if a != Nat.0 {
        gcd_nonzero_left(a, b)
        a.gcd(b) != Nat.0
        gcd_divides_right(a, b)
        let k: Nat satisfy { a.gcd(b) * k = b }
        gcd_divides_left(a, b)
        let q: Nat satisfy { a.gcd(b) * q = a }
        gcd_mul_lcm(a, b)
        a.gcd(b) * a.lcm(b) = a * b
        a * b = q * a.gcd(b) * b
        a.gcd(b) * a.lcm(b) = q * a.gcd(b) * b
        q * a.gcd(b) * b = a.gcd(b) * (q * b)
        a.gcd(b) * a.lcm(b) = a.gcd(b) * (q * b)
        a.lcm(b) = q * b
        b = k * a.gcd(b)
        q * b = q * (k * a.gcd(b))
        q * (k * a.gcd(b)) = (k * q) * a.gcd(b)
        q * a.gcd(b) = a
        (k * q) * a.gcd(b) = k * (q * a.gcd(b))
        k * (q * a.gcd(b)) = k * a
        q * b = k * a
        a.lcm(b) = k * a
        a * k = k * a
        a * k = a.lcm(b)
        a.divides(a.lcm(b))
    }
}

/// Local Nat helper: the right input divides the lcm.
theorem nat_lcm_divides_right_for_int_lcm(a: Nat, b: Nat) {
    b.divides(a.lcm(b))
} by {
    if b = Nat.0 {
        nat_lcm_zero_right_for_int_lcm(a)
        b * Nat.0 = Nat.0
        b.divides(a.lcm(b))
    }
    if b != Nat.0 {
        gcd_comm(a, b)
        gcd_nonzero_left(b, a)
        a.gcd(b) != Nat.0
        gcd_divides_left(a, b)
        let q: Nat satisfy { a.gcd(b) * q = a }
        gcd_mul_lcm(a, b)
        a.gcd(b) * a.lcm(b) = a * b
        a * b = q * a.gcd(b) * b
        a.gcd(b) * a.lcm(b) = q * a.gcd(b) * b
        q * a.gcd(b) * b = a.gcd(b) * (q * b)
        a.gcd(b) * a.lcm(b) = a.gcd(b) * (q * b)
        a.lcm(b) = q * b
        b * q = q * b
        b * q = a.lcm(b)
        b.divides(a.lcm(b))
    }
}

/// Local Nat helper: Euclid's lemma for coprime natural numbers.
theorem nat_coprime_divides_of_divides_mul_for_int_lcm(a: Nat, b: Nat, c: Nat) {
    a.coprime(b) and a.divides(b * c) implies a.divides(c)
} by {
    if a.coprime(b) and a.divides(b * c) {
        a.gcd(b) = Nat.1
        gcd_mult_right(a, b, c)
        a.gcd(b) * c = (a * c).gcd(b * c)
        Nat.1 * c = (a * c).gcd(b * c)
        c = (a * c).gcd(b * c)
        a.divides(a * c)
        a.divides(b * c)
        divides_gcd(a, a * c, b * c)
        a.divides((a * c).gcd(b * c))
        a.divides(c)
    }
}

/// Local Nat helper: common multiples of `0` are divisible by `0.lcm(b)`.
theorem nat_lcm_divides_of_common_left_zero_for_int_lcm(b: Nat, c: Nat) {
    Nat.0.divides(c) implies Nat.0.lcm(b).divides(c)
} by {
    if Nat.0.divides(c) {
        let z: Nat satisfy { Nat.0 * z = c }
        c = Nat.0
        nat_lcm_zero_left_for_int_lcm(b)
        Nat.0.lcm(b) = Nat.0
        Nat.0 * Nat.0 = Nat.0
        Nat.0.lcm(b).divides(c)
    }
}

/// Local Nat helper: common multiples of `0` are divisible by `a.lcm(0)`.
theorem nat_lcm_divides_of_common_right_zero_for_int_lcm(a: Nat, c: Nat) {
    Nat.0.divides(c) implies a.lcm(Nat.0).divides(c)
} by {
    if Nat.0.divides(c) {
        let z: Nat satisfy { Nat.0 * z = c }
        c = Nat.0
        nat_lcm_zero_right_for_int_lcm(a)
        a.lcm(Nat.0) = Nat.0
        Nat.0 * Nat.0 = Nat.0
        a.lcm(Nat.0).divides(c)
    }
}

/// Local Nat helper: every nonzero common multiple is divisible by the lcm.
theorem nat_lcm_divides_of_common_nonzero_for_int_lcm(a: Nat, b: Nat, c: Nat) {
    a != Nat.0 and b != Nat.0 and a.divides(c) and b.divides(c)
        implies a.lcm(b).divides(c)
} by {
    if a != Nat.0 and b != Nat.0 and a.divides(c) and b.divides(c) {
        gcd_nonzero_left(a, b)
        a.gcd(b) != Nat.0
        gcd_divides_left(a, b)
        gcd_divides_right(a, b)
        let q: Nat satisfy { a.gcd(b) * q = a }
        let k: Nat satisfy { a.gcd(b) * k = b }
        q * a.gcd(b) = a.gcd(b) * q
        k * a.gcd(b) = a.gcd(b) * k
        q * a.gcd(b) = a
        k * a.gcd(b) = b
        cofactor(a, b, q, k)
        q.gcd(k) = Nat.1
        gcd_mul_lcm(a, b)
        a.gcd(b) * a.lcm(b) = a * b
        a * b = q * a.gcd(b) * b
        a.gcd(b) * a.lcm(b) = q * a.gcd(b) * b
        q * a.gcd(b) * b = a.gcd(b) * (q * b)
        a.gcd(b) * a.lcm(b) = a.gcd(b) * (q * b)
        a.lcm(b) = q * b
        let x: Nat satisfy { a * x = c }
        let y: Nat satisfy { b * y = c }
        b * y = a * x
        b * y = q * a.gcd(b) * x
        b = k * a.gcd(b)
        k * a.gcd(b) * y = q * a.gcd(b) * x
        k * a.gcd(b) * y = a.gcd(b) * (k * y)
        q * a.gcd(b) * x = a.gcd(b) * (q * x)
        a.gcd(b) * (k * y) = a.gcd(b) * (q * x)
        k * y = q * x
        q.divides(k * y)
        q.coprime(k)
        nat_coprime_divides_of_divides_mul_for_int_lcm(q, k, y)
        q.divides(y)
        let z: Nat satisfy { q * z = y }
        q * b * z = b * (q * z)
        q * b * z = b * y
        q * b * z = c
        a.lcm(b) * z = c
        a.lcm(b).divides(c)
    }
}

/// Local Nat helper: every common multiple is divisible by the lcm.
theorem nat_lcm_divides_of_common_for_int_lcm(a: Nat, b: Nat, c: Nat) {
    a.divides(c) and b.divides(c) implies a.lcm(b).divides(c)
} by {
    if a.divides(c) and b.divides(c) {
        if a = Nat.0 {
            nat_lcm_divides_of_common_left_zero_for_int_lcm(b, c)
            Nat.0.lcm(b).divides(c)
            a.lcm(b).divides(c)
        } else {
            if b = Nat.0 {
                nat_lcm_divides_of_common_right_zero_for_int_lcm(a, c)
                a.lcm(Nat.0).divides(c)
                a.lcm(b).divides(c)
            } else {
                nat_lcm_divides_of_common_nonzero_for_int_lcm(a, b, c)
                a.lcm(b).divides(c)
            }
        }
    }
}

/// The left input divides the integer lcm.
theorem int_lcm_divides_left(a: Int, b: Int) {
    a.divides(a.lcm(b))
} by {
    nat_lcm_divides_left_for_int_lcm(abs(a), abs(b))
    abs(a).divides(abs(a).lcm(abs(b)))
    div_from_nat(abs(a), abs(a).lcm(abs(b)))
    Int.from_nat(abs(a)).divides(Int.from_nat(abs(a).lcm(abs(b))))
    a.divides(Int.from_nat(abs(a)))
    a.lcm(b) = Int.from_nat(abs(a).lcm(abs(b)))
    div_trans(a, Int.from_nat(abs(a)), a.lcm(b))
    a.divides(a.lcm(b))
}

/// The right input divides the integer lcm.
theorem int_lcm_divides_right(a: Int, b: Int) {
    b.divides(a.lcm(b))
} by {
    nat_lcm_divides_right_for_int_lcm(abs(a), abs(b))
    abs(b).divides(abs(a).lcm(abs(b)))
    div_from_nat(abs(b), abs(a).lcm(abs(b)))
    Int.from_nat(abs(b)).divides(Int.from_nat(abs(a).lcm(abs(b))))
    b.divides(Int.from_nat(abs(b)))
    a.lcm(b) = Int.from_nat(abs(a).lcm(abs(b)))
    div_trans(b, Int.from_nat(abs(b)), a.lcm(b))
    b.divides(a.lcm(b))
}

/// Universal property: every common integer multiple is divisible by the lcm.
theorem int_lcm_divides_of_common(a: Int, b: Int, c: Int) {
    a.divides(c) and b.divides(c) implies a.lcm(b).divides(c)
} by {
    if a.divides(c) and b.divides(c) {
        div_imp_div_abs(a, c)
        abs(a).divides(abs(c))
        div_imp_div_abs(b, c)
        abs(b).divides(abs(c))
        nat_lcm_divides_of_common_for_int_lcm(abs(a), abs(b), abs(c))
        abs(a).lcm(abs(b)).divides(abs(c))
        a.lcm(b) = Int.from_nat(abs(a).lcm(abs(b)))
        abs(a.lcm(b)) = abs(a).lcm(abs(b))
        abs(a.lcm(b)).divides(abs(c))
        div_abs_imp_div(a.lcm(b), c)
        a.lcm(b).divides(c)
    }
}

/// Integer lcm divisibility characterizes common integer multiples.
theorem int_lcm_divides_iff(a: Int, b: Int, c: Int) {
    a.lcm(b).divides(c) = (a.divides(c) and b.divides(c))
} by {
    if a.lcm(b).divides(c) {
        int_lcm_divides_left(a, b)
        a.divides(a.lcm(b))
        div_trans(a, a.lcm(b), c)
        a.divides(c)
        int_lcm_divides_right(a, b)
        b.divides(a.lcm(b))
        div_trans(b, a.lcm(b), c)
        b.divides(c)
        a.divides(c) and b.divides(c)
    }
    if a.divides(c) and b.divides(c) {
        int_lcm_divides_of_common(a, b, c)
        a.lcm(b).divides(c)
    }
}
