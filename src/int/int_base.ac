from algebra.add import Add
from algebra.mul import Mul
from nat import Nat, alt_suc_ne_zero, lte_antisymm
from algebra.neg import Neg

/// The `Int` type represents integers.
/// It's defined by its two constructors.
///
/// `from_nat` takes a natural number to an integer, which seems intuitive.
/// `neg_suc` takes `x` to `-(x+1)`, which is somewhat less intuitive. We do this so
/// that every integer can be represented either as a `from_nat` or a `neg_suc`.
///
/// ```acorn
/// numerals Int
///
/// 2 = Int.from_nat(Nat.2)
/// -2 = Int.neg_suc(Nat.1)
/// ```
inductive Int {
    /// `Int.from_nat` converts a natural number to an integer via the typical embedding.
    from_nat(Nat)

    /// `Int.neg_suc` converts a natural number `x` into `-(x+1)`.
    /// This isn't particularly intuitive, it's just to give every integer a unique constructor.
    /// In particular, `neg_suc` can construct any negative integer, but not zero.
    neg_suc(Nat)
}

let from_nat = Int.from_nat

/// Yields the absolute value of an integer as a natural number.
define abs(a: Int) -> Nat {
    match a {
        Int.from_nat(n) {
            n
        }
        Int.neg_suc(k) {
            k.suc
        }
    }
}

theorem abs_from_nat(n: Nat) {
    abs(Int.from_nat(n)) = n
}

attributes Int {
    /// The integer two.
    let 2: Int = Int.from_nat(Nat.2)
    /// The integer three.
    let 3: Int = Int.from_nat(Nat.3)
    /// The integer four.
    let 4: Int = Int.from_nat(Nat.4)
    /// The integer five.
    let 5: Int = Int.from_nat(Nat.5)
    /// The integer six.
    let 6: Int = Int.from_nat(Nat.6)
    /// The integer seven.
    let 7: Int = Int.from_nat(Nat.7)
    /// The integer eight.
    let 8: Int = Int.from_nat(Nat.8)
    /// The integer nine.
    let 9: Int = Int.from_nat(Nat.9)
    /// The integer ten.
    let 10: Int = Int.from_nat(Nat.10)
}

from algebra.zero import Zero

instance Int: Zero {
    let 0: Int = Int.from_nat(Nat.0)
}

from algebra.one import One

instance Int: One {
    let 1: Int = Int.from_nat(Nat.1)
}

numerals Int

/// Converts a natural number to its negative integer equivalent.
define neg_nat(n: Nat) -> Int {
    if n = Nat.0 {
        0
    } else {
        Int.neg_suc(n - Nat.1)
    }
}

theorem neg_nat_zero { neg_nat(Nat.0) = 0 }

theorem neg_nat_suc(n: Nat) {
    neg_nat(n.suc) = Int.neg_suc(n)
}

theorem abs_neg_nat(n: Nat) {
    abs(neg_nat(n)) = n
} by {
    if n = Nat.0 {
    } else {
        let k: Nat satisfy { k.suc = n }
        abs(Int.neg_suc(k)) = k.suc
        abs(neg_nat(n)) = n
    }
}

/// The negation of an integer.
instance Int: Neg {
    define neg(self) -> Int {
        match self {
            Int.from_nat(n) {
                neg_nat(n)
            }
            Int.neg_suc(n) {
                Int.from_nat(n.suc)
            }
        }
    }
}

theorem neg_zero {
    -0 = 0
}

theorem neg_neg_suc(n: Nat) {
    -Int.neg_suc(n) = Int.from_nat(n.suc)
}

theorem neg_from_nat(n: Nat) {
    -Int.from_nat(n) = neg_nat(n)
}

theorem neg_neg(a: Int) {
    -(-a) = a
} by {
    match a {
        Int.from_nat(n) {
            if n = Nat.0 {
                -(-a) = a
            } else {
                let k: Nat satisfy { k.suc = n }
                Int.neg_suc(n - Nat.1) = neg_nat(n)
                -Int.neg_suc(k) = Int.from_nat(k.suc)
                -(-a) = a
            }
        }
        Int.neg_suc(n) {
        }
    }
}

theorem fix_neg(a: Int) {
    -a = a implies a = 0
} by {
    if -a = a {
        match a {
            Int.from_nat(n) {
                n = Nat.0
                a = 0
            }
            Int.neg_suc(n) {
            }
        }
    }
}

theorem abs_neg(a: Int) {
    abs(-a) = abs(a)
} by {
    match a {
        Int.from_nat(n) {
        }
        Int.neg_suc(n) {
            abs(-a) = abs(a)
        }
    }
}

theorem neg_or_pos(a: Int) {
    a = Int.from_nat(abs(a)) or a = -(Int.from_nat(abs(a)))
} by {
    match a {
        Int.from_nat(n) {
        }
        Int.neg_suc(n) {
            a = -(Int.from_nat(abs(a)))
        }
    }
}

theorem from_eq_neg_from(p: Nat, q: Nat) {
    Int.from_nat(p) = -(Int.from_nat(q)) implies p = Nat.0 and q = Nat.0
} by {
    abs(-Int.from_nat(q)) = abs(Int.from_nat(q))
    abs(Int.from_nat(q)) = q
    abs(Int.from_nat(Nat.0)) = Nat.0
    Int.from_nat(p) = 0
}

theorem abs_zero { abs(0) = Nat.0 }

theorem one_neq_zero { 1 != 0 }

// Subtraction that goes from naturals into integers.
// We will use this as the primary representation for proving things about integers, so we prove
// as many useful things about sub_nat as we can, before defining more stuff.
define sub_nat(m: Nat, n: Nat) -> Int {
    if n <= m {
        Int.from_nat(m - n)
    } else {
        -(Int.from_nat(n - m))
    }
}

theorem sub_nat_zero_right(n: Nat) {
    sub_nat(n, Nat.0) = Int.from_nat(n)
} by {
}

theorem sub_nat_zero_left(n: Nat) { sub_nat(Nat.0, n) = -(Int.from_nat(n)) } by {
    -Int.from_nat(n - Nat.0) = sub_nat(Nat.0, n) or n <= Nat.0
    sub_nat(Nat.0, Nat.0) = Int.from_nat(Nat.0)
    not n <= Nat.0 or n = Nat.0
    n - Nat.0 = n
}

theorem sub_nat_self(n: Nat) { sub_nat(n, n) = 0 }

theorem sub_nat_add_left(p: Nat, q: Nat) {
    sub_nat(p + q, q) = Int.from_nat(p)
}

theorem neg_sub_nat(m: Nat, n: Nat) { sub_nat(m, n) = -(sub_nat(n, m)) } by {
    if m = n {
    } else {
        if n <= m {
            sub_nat(m, n) = -(sub_nat(n, m))
        } else {
            m < n
            m <= n
            Int.from_nat(n - m) = sub_nat(n, m)
            sub_nat(m, n) = -(sub_nat(n, m))
        }
    }
}

theorem sub_nat_add_right(p: Nat, q: Nat) { sub_nat(p, p + q) = -(Int.from_nat(q)) }

// Half of a "without loss of generality" argument
theorem sub_nat_eq_helper(m: Nat, n: Nat, p: Nat, q: Nat) {
    m + n = p + q and p <= m implies sub_nat(m, p) = sub_nat(q, n)
} by {
    let (d: Nat) satisfy { p + d = m }
    d + p = p + d
    m - p = d
    Int.from_nat(m - p) = sub_nat(m, p)
    sub_nat(d + n, n) = Int.from_nat(d)
    q = d + n
}

theorem sub_nat_eq(m: Nat, n: Nat, p: Nat, q: Nat) {
    m + n = p + q implies sub_nat(m, p) = sub_nat(q, n)
} by {
    -sub_nat(n, q) = sub_nat(q, n)
    -sub_nat(p, m) = sub_nat(m, p)
    m <= p or p < m
    if p <= m {
    } else {
        m <= p
        sub_nat(n, q) = sub_nat(p, m)
    }
}

theorem sub_nat_imp_add(i: Nat, j: Nat, k: Nat) {
    sub_nat(i, j) = Int.from_nat(k) implies j + k = i
} by {
    if j <= i {
        j + (i - j) = i - j + j
    } else {
        let (d: Nat) satisfy { i + d = j }
        sub_nat(i, i + d) = -Int.from_nat(d)
        -Int.from_nat(j - i) != Int.from_nat(k) or k = Nat.0
        false
    }
}

theorem sub_nat_negate_imp_add(i: Nat, j: Nat, k: Nat) {
    sub_nat(i, j) = -(Int.from_nat(k)) implies i + k = j
} by {
    if j <= i {
        let (d: Nat) satisfy { j + d = i }
    } else {
    }
}

theorem sub_nat_cancel_right(i: Nat, j: Nat, k: Nat) { sub_nat(i, k) = sub_nat(j, k) implies i = j } by {
    if k <= i {
        let (d: Nat) satisfy { k + d = i }
        sub_nat(i, k) = Int.from_nat(d)
        k + d = j
        i = j
    } else {
        let (d: Nat) satisfy { i + d = k }
        sub_nat(i, i + d) = -Int.from_nat(d)
        j + d = k
        k - d = j
        i = j
    }
}

theorem sub_nat_cancel_left(i: Nat, j: Nat, k: Nat) {
    sub_nat(k, i) = sub_nat(k, j) implies i = j
} by {
    sub_nat(i, k) = sub_nat(j, k)
}

theorem sub_nat_add_cancel_right(m: Nat, n: Nat, k: Nat) {
    sub_nat(m, n) = sub_nat(m + k, n + k)
}

theorem sub_nat_add_cancel_left(m: Nat, n: Nat, k: Nat) {
    sub_nat(m, n) = sub_nat(k + m, k + n)
}

// Half of a "without loss of generality" argument
theorem sub_nat_imp_add_eq_helper(m: Nat, n: Nat, p: Nat, q: Nat) {
    sub_nat(m, p) = sub_nat(q, n) and p <= m implies m + n = p + q
} by {
    let (d: Nat) satisfy { p + d = m }
    sub_nat(d, Nat.0) = Int.from_nat(d)
    p + (d + n) = p + d + n
    d + n = n + d
    n + d = q
}

theorem sub_nat_imp_add_eq(m: Nat, n: Nat, p: Nat, q: Nat) { sub_nat(m, p) = sub_nat(q, n) implies m + n = p + q } by {
    if p <= m {
    } else {
        m <= p
    }
}

theorem sub_nat_double_cancel_left(p: Nat, q: Nat, r: Nat, s: Nat, t: Nat) {
    sub_nat(p + t, q) = sub_nat(r + t, s) implies sub_nat(p, q) = sub_nat(r, s)
}

theorem sub_nat_double_cancel_right(p: Nat, q: Nat, r: Nat, s: Nat, t: Nat) {
    sub_nat(p, q + t) = sub_nat(r, s + t) implies sub_nat(p, q) = sub_nat(r, s)
}

// Now that we've proven a bunch of stuff about sub_nat, we define the positive and negative parts so that we can
// represent each integer as a sub_nat, and start defining useful functions on integers.

attributes Int {
    /// True if the integer is negative.
    define is_negative(self) -> Bool {
        self != Int.from_nat(abs(self))
    }

    /// True if the integer is positive.
    define is_positive(self) -> Bool {
        (-self).is_negative
    }
}

theorem zero_not_neg { not 0.is_negative }

theorem zero_not_pos {
    not 0.is_positive
}

theorem one_pos { 1.is_positive }

theorem nonzero_pos_or_neg(a: Int) {
    a != 0 implies a.is_positive or a.is_negative
} by {
    Int.from_nat(abs(-a)) = -a or (-a).is_negative
    abs(-a) = abs(a)
    -a != a or 0 = a
}

theorem pos_is_not_neg(a: Int) { a.is_positive implies not a.is_negative }

theorem non_pos_is_neg_abs(a: Int) { not a.is_positive implies a = -(Int.from_nat(abs(a))) }

attributes Int {
    /// The positive part of this integer.
    define pos_part(self) -> Nat {
        if self.is_positive {
            abs(self)
        } else {
            Nat.0
        }
    }

    /// The negative part of this integer.
    define neg_part(self) -> Nat {
        if self.is_positive {
            Nat.0
        } else {
            abs(self)
        }
    }
}

theorem sub_nat_parts(a: Int) { sub_nat(a.pos_part, a.neg_part) = a } by {
    if a.is_positive {
    } else {
        sub_nat(a.pos_part, a.neg_part) = a
    }
}

theorem pos_part_neg(a: Int) { (-a).pos_part = a.neg_part } by {
    if a.is_positive {
    } else {
        (-a).pos_part = abs(a)
        (-a).pos_part = a.neg_part
    }
}

theorem pos_part_from(n: Nat) { Int.from_nat(n).pos_part = n } by {
    if n = Nat.0 {
    } else {
        Int.from_nat(n).pos_part = n
    }
}

theorem neg_part_from(n: Nat) {
    Int.from_nat(n).neg_part = Nat.0
} by {
    Int.from_nat(n).pos_part = n
    Int.from_nat(n).pos_part = Nat.0 or Int.from_nat(n).is_positive
    if Int.from_nat(n).is_positive {
    } else {
    }
}

theorem neg_part_neg(a: Int) { (-a).neg_part = a.pos_part }

theorem parts_sub_nat(j: Nat, k: Nat) { sub_nat(j, k).pos_part + k = sub_nat(j, k).neg_part + j }

theorem add_part_sub_nat(r: Nat, s: Nat) { r + sub_nat(r, s).neg_part = s + sub_nat(r, s).pos_part }

// Addition, and theorems about addition

/// The sum of two integers.
instance Int: Add {
    define add(self, other: Int) -> Int {
        sub_nat(self.pos_part + other.pos_part, self.neg_part + other.neg_part)
    }
}

theorem add_zero_left(a: Int) { (0 + a) = a } by {
}

theorem add_zero_right(a: Int) { a + 0 = a }

theorem add_comm(a: Int, b: Int) { a + b = b + a }

theorem neg_distrib(a: Int, b: Int) { -(a + b) = -a + -b }

theorem parts_of_add(a: Int, b: Int) {
    (a.pos_part + b.pos_part + (a + b).neg_part =
     a.neg_part + b.neg_part + (a + b).pos_part)
} by {
    let j = a.pos_part + b.pos_part
    let k = a.neg_part + b.neg_part
}

theorem add_neg(a: Int) { a + -a = 0 }

theorem add_eq_zero(a: Int, b: Int) { a + b = 0 implies a = -b } by {
    sub_nat(a.pos_part + b.pos_part, a.neg_part + b.neg_part) = a + b
    sub_nat(a.pos_part + b.pos_part, a.neg_part + b.neg_part) != Int.from_nat(Nat.0) or
        a.neg_part + b.neg_part + Nat.0 = a.pos_part + b.pos_part
    (-b).neg_part = b.pos_part
    (-b).pos_part = b.neg_part
    a.neg_part + (-b).pos_part + Nat.0 = a.neg_part + (-b).pos_part
    a.pos_part + (-b).neg_part = a.neg_part + (-b).pos_part
}

theorem add_right_cancel(a: Int, b: Int, c: Int) { a + c = b + c implies a = b } by {
    sub_nat(a.pos_part, a.neg_part + c.neg_part) = sub_nat(b.pos_part, b.neg_part + c.neg_part)
}

theorem add_left_cancel(a: Int, b: Int, c: Int) { c + a = c + b implies a = b }

theorem add_sub_nat_left_pos(p: Nat, q: Nat, r: Nat) {
    (sub_nat(p, q) + Int.from_nat(r)) = sub_nat(p + r, q)
} by {
    sub_nat(p + r, q) = sub_nat(sub_nat(p, q).pos_part + r, sub_nat(p, q).neg_part)
}

theorem add_sub_nat_left_neg(p: Nat, q: Nat, r: Nat) {
    (sub_nat(p, q) + -(Int.from_nat(r))) = sub_nat(p, q + r)
}

theorem add_sub_nat_left(p: Nat, q: Nat, a: Int) {
    (sub_nat(p, q) + a) = sub_nat(p + a.pos_part, q + a.neg_part)
} by {
    if a.is_positive {
    } else {
        (sub_nat(p, q) + a) = sub_nat(p + a.pos_part, q + a.neg_part)
    }
}

theorem add_sub_nat_right(p: Nat, q: Nat, a: Int) {
    a + sub_nat(p, q) = sub_nat(a.pos_part + p, a.neg_part + q)
}

theorem add_sub_nat(p: Nat, q: Nat, r: Nat, s: Nat) {
    (sub_nat(p, q) + sub_nat(r, s)) = sub_nat(p + r, q + s)
} by {
    s + sub_nat(r, s).pos_part = r + sub_nat(r, s).neg_part
    p + q + (r + sub_nat(r, s).neg_part) = p + r + (q + sub_nat(r, s).neg_part)
    q + s + (p + sub_nat(r, s).pos_part) = q + p + (s + sub_nat(r, s).pos_part)
    q + p = p + q
    sub_nat(p + r, q + s) = sub_nat(p + sub_nat(r, s).pos_part,
                                                    q + sub_nat(r, s).neg_part)
}

theorem add_sub_nat_3_left(p: Nat, q: Nat, r: Nat, s: Nat, t: Nat, u: Nat) {
    sub_nat(p, q) + sub_nat(r, s) + sub_nat(t, u) = sub_nat(p + r + t, q + s + u)
} by {
    let lhs = sub_nat(p, q) + sub_nat(r, s) + sub_nat(t, u)
}

theorem add_sub_nat_3_right(p: Nat, q: Nat, r: Nat, s: Nat, t: Nat, u: Nat) {
    sub_nat(p, q) + (sub_nat(r, s) + sub_nat(t, u)) = sub_nat(p + (r + t), q + (s + u))
} by {
    let lhs = sub_nat(p, q) + (sub_nat(r, s) + sub_nat(t, u))
}

theorem add_sub_nat_assoc(p: Nat, q: Nat, r: Nat, s: Nat, t: Nat, u: Nat) {
    sub_nat(p, q) + sub_nat(r, s) + sub_nat(t, u) = sub_nat(p, q) + (sub_nat(r, s) + sub_nat(t, u))
} by {
    let lhs = sub_nat(p, q) + sub_nat(r, s) + sub_nat(t, u)
}

theorem add_assoc(a: Int, b: Int, c: Int) { a + (b + c) = a + b + (c) } by {
    sub_nat(a.pos_part, a.neg_part) + sub_nat(b.pos_part, b.neg_part) + sub_nat(c.pos_part, c.neg_part) = sub_nat(a.pos_part, a.neg_part) + (sub_nat(b.pos_part, b.neg_part) + sub_nat(c.pos_part, c.neg_part))
}

theorem add_from_nat(a: Nat, b: Nat) {
    Int.from_nat(a) + Int.from_nat(b) = Int.from_nat(a + b)
}

theorem add_pos_nonneg(a: Int, b: Int) {
    a.is_positive and not b.is_negative implies (a + b).is_positive
} by {
    if b = 0 {
    } else {
        Int.from_nat(abs(a) + abs(b)).pos_part = abs(a) + abs(b)
        Int.from_nat(abs(a) + abs(b)).neg_part = Nat.0
        Int.from_nat(abs(a) + abs(b)).pos_part = Nat.0 or Int.from_nat(abs(a) + abs(b)).is_positive
        (Int.from_nat(abs(a) + abs(b))).is_positive
        (a + b).is_positive
    }
}

theorem add_neg_nonpos(a: Int, b: Int) {
    a.is_negative and not b.is_positive implies (a + b).is_negative
} by {
    -a + -b = -(a + b)
    (-b).is_negative = b.is_positive
    (--a).is_negative = (-a).is_positive
    --a = a
    --(a + b) = a + b
    (-(-a + -b)).is_negative = (-a + -b).is_positive
}

theorem add_nonneg_nonneg(a: Int, b: Int) {
    not a.is_negative and not b.is_negative implies not (a + b).is_negative
}

theorem add_nonpos_nonpos(a: Int, b: Int) {
    not a.is_positive and not b.is_positive implies not (a + b).is_positive
} by {
    a.is_positive or a.is_negative or 0 = a
    0 + b = b
    not (a + b).is_positive or not (a + b).is_negative
}

theorem add_comm_4(a: Int, b: Int, c: Int, d: Int) { (a + b) + (c + d) = (a + c) + (b + d) }

// Connecting Int to its additive algebraic structure.

from algebra.add_semigroup import AddSemigroup

instance Int: AddSemigroup

from algebra.add_comm_semigroup import AddCommSemigroup

instance Int: AddCommSemigroup

from algebra.add_monoid import AddMonoid

instance Int: AddMonoid

from algebra.add_comm_monoid import AddCommMonoid

instance Int: AddCommMonoid

from algebra.add_group import AddGroup

instance Int: AddGroup

from algebra.add_comm_group import AddCommGroup

instance Int: AddCommGroup

// Subtraction, and theorems about subtraction

theorem sub_zero_right(a: Int) { a - 0 = a }

theorem sub_zero_left(a: Int) { 0 - a = -a }

theorem sub_anticomm(a: Int, b: Int) { a - b = -(b - a) }

theorem sub_self(a: Int) { a - a = 0 }

theorem sub_eq_zero(a: Int, b: Int) { a - b = 0 implies a = b }

theorem sub_add_left(a: Int, b: Int) {
    (a + b) - b = a
}

theorem neg_sub(a: Int, b: Int) { -(a - b) = b - a }

theorem sub_add_right(a: Int, b: Int) { a - (a + b) = -b }

theorem sub_imp_add(a: Int, b: Int, c: Int) { a - b = c implies b + c = a }

theorem sub_negate_imp_add(a: Int, b: Int, c: Int) { a - b = -(c) implies a + c = b }

theorem sub_cancel_right(a: Int, b: Int, c: Int) { a - c = b - c implies a = b }

theorem sub_cancel_left(a: Int, b: Int, c: Int) { a - b = a - c implies b = c }

theorem sub_add_cancel_left(a: Int, b: Int, c: Int) { (a + b) - (a + c) = b - c } by {
    a + b + (-a + -c) = a + -a + (b + -c)
    -a + -c = -(a + c)
    a + -a = Int.0
    Int.0 + (b - c) = b - c
}

theorem sub_add_cancel_right(a: Int, b: Int, c: Int) { (a + c) - (b + c) = a - b }

// Integer-natural multiplication

attributes Int {
    /// Multiply this integer by a natural number.
    define mul_nat(self, n: Nat) -> Int {
        if self.is_negative {
            -(Int.from_nat(abs(self) * n))
        } else {
            Int.from_nat(abs(self) * n)
        }
    }
}

theorem mul_nat_zero_right(a: Int) { a.mul_nat(Nat.0) = Int.0 } by {
    match a {
        Int.from_nat(n) {
            Int.from_nat(n * Nat.0) = Int.from_nat(n).mul_nat(Nat.0)
            a.mul_nat(Nat.0) = Int.0
        }
        Int.neg_suc(n) {
            -Int.from_nat(n.suc * Nat.0) = Int.neg_suc(n).mul_nat(Nat.0)
            a.mul_nat(Nat.0) = Int.0
        }
    }
}

theorem mul_nat_zero_left(n: Nat) { 0.mul_nat(n) = 0 } by {
    Int.from_nat(Nat.mul(Nat.0, n)) = 0.mul_nat(n)
}

theorem mul_nat_nonpos_left(a: Int, n: Nat) {
    not a.is_positive implies a.mul_nat(n) = -(Int.from_nat(abs(a) * n))
} by {
    a.is_positive or a.is_negative or Int.0 = a
    Int.0.mul_nat(n) = Int.0
    Nat.0 * n = Nat.0
    if a = 0 {
    } else {
    }
}

theorem mul_nat_negate_left(a: Int, n: Nat) { (-a).mul_nat(n) = -(a.mul_nat(n)) } by {
    (-a).is_negative = a.is_positive
    abs(-a) = abs(a)
    not a.is_positive or not a.is_negative
    Nat.0 + n = n
    if a.is_positive {
    } else {
        not (-a).is_negative
        (-a).mul_nat(n) = -(a.mul_nat(n))
    }
}

theorem mul_nat_nonneg_suc(a: Int, n: Nat) {
    not a.is_negative implies a.mul_nat(n.suc) = a.mul_nat(n) + a
} by {
    a.mul_nat(n.suc) = Int.from_nat(abs(a) * n) +  Int.from_nat(abs(a))
}

theorem mul_nat_suc(a: Int, n: Nat) {
    a.mul_nat(n.suc) = (a.mul_nat(n) + a)
} by {
    if a.is_negative {
        a.mul_nat(n.suc) = -((-a).mul_nat(n) + -a)
    } else {
    }
}

theorem mul_nat_distrib_right(a: Int, b: Int, n: Nat) {
    (a + b).mul_nat(n) = (a.mul_nat(n) + b.mul_nat(n))
} by {
    define f(x: Nat) -> Bool {
        (a + b).mul_nat(x) = (a.mul_nat(x) + b.mul_nat(x))
    }

    // Prove the base case
    (a + b).mul_nat(Nat.0) = Int.0
    a.mul_nat(Nat.0) = Int.0
    b.mul_nat(Nat.0) = Int.0
    Int.0 + b.mul_nat(Nat.0) = b.mul_nat(Nat.0)
    a.mul_nat(Nat.0) + b.mul_nat(Nat.0) != (a + b).mul_nat(Nat.0) or f(Nat.0)
    f(Nat.0)
    f(Nat.0)

    // Induct
    forall(x: Nat) {
        if f(x) {
            not f(x) or a.mul_nat(x) + b.mul_nat(x) = (a + b).mul_nat(x)
            (a + b).mul_nat(x.suc) = (a.mul_nat(x) + a) + (b.mul_nat(x) + b)
            f(x.suc)
        }
    }
    f(n)
}

theorem mul_nat_from_nat_left(a: Nat, b: Nat) { Int.from_nat(a).mul_nat(b) = Int.from_nat(a * b) } by {
    if a = Nat.0 {
    } else {
    }
}

// Integer-integer multiplication

/// The product of two integers.
instance Int: Mul {
    define mul(self, n: Int) -> Int {
        if n.is_positive {
            self.mul_nat(abs(n))
        } else {
            -(self.mul_nat(abs(n)))
        }
    }
}

attributes Int {
    /// The integer formed by appending a digit to this integer in base 10.
    define read(self, other: Int) -> Int { 10 * self + other }
}

theorem mul_zero_right(a: Int) { a * 0 = 0 } by {
    a * 0 = -(a.mul_nat(Nat.0))
}

theorem mul_nat_from_nat_right(a: Int, n: Nat) { a.mul_nat(n) = (a * Int.from_nat(n)) } by {
    if n = Nat.0 {
    } else {
        a * Int.from_nat(n) = a.mul_nat(abs(Int.from_nat(n)))
    }
}

theorem mul_nonneg_right(a: Int, b: Int) { not b.is_negative implies a * b = a.mul_nat(abs(b)) } by {
    if b = 0 {
    } else {
    }
}

theorem mul_nonneg_nonneg(a: Int, b: Int) {
    not a.is_negative and not b.is_negative implies a * b = Int.from_nat(abs(a) * abs(b))
}

theorem mul_nonneg_nonpos(a: Int, b: Int) {
    not a.is_negative and not b.is_positive implies a * b = -(Int.from_nat(abs(a) * abs(b)))
}

theorem mul_nonpos_nonneg(a: Int, b: Int) {
    not a.is_positive and not b.is_negative implies a * b = -(Int.from_nat(abs(a) * abs(b)))
}

theorem mul_nonpos_nonpos(a: Int, b: Int) {
    not a.is_positive and not b.is_positive implies a * b = Int.from_nat(abs(a) * abs(b))
}

theorem mul_nonneg_nonneg_nonneg(a: Int, b: Int) {
    not a.is_negative and not b.is_negative implies not (a * b).is_negative
} by {
    Int.from_nat(abs(a)).mul_nat(abs(b)) = Int.from_nat(abs(a) * abs(b))
    Int.from_nat(abs(a)) = a or a.is_negative
    Int.from_nat(abs(b)) = b or b.is_negative
}

theorem mul_nonneg_nonpos_nonpos(a: Int, b: Int) {
    not a.is_negative and not b.is_positive implies not (a * b).is_positive
} by {
    -Int.from_nat(abs(a) * abs(b)) = a * b or b.is_positive or a.is_negative
    abs(b) * abs(a) = abs(a) * abs(b)
    -Int.from_nat(abs(b) * abs(a)) = neg_nat(abs(b) * abs(a))
    abs(neg_nat(abs(b) * abs(a))) = abs(b) * abs(a)
}

theorem mul_nonpos_nonneg_nonpos(a: Int, b: Int) {
    not a.is_positive and not b.is_negative implies not (a * b).is_positive
}

theorem mul_nonpos_nonpos_nonneg(a: Int, b: Int) {
    not a.is_positive and not b.is_positive implies not (a * b).is_negative
} by {
    -Int.from_nat(abs(a) * abs(b)) = a.mul_nat(abs(b)) or a.is_positive
    -(--a).mul_nat(abs(b)) = --a * b or b.is_positive
}

theorem mul_zero_left(a: Int) {
    0 * a = 0
} by {
}

theorem mul_comm(a: Int, b: Int) { a * b = b * a } by {
    if a.is_positive {
        if b.is_positive {
            b * a = Int.from_nat(abs(b) * abs(a))
            a * b = b * a
        } else {
            b * a = -(Int.from_nat(abs(b) * abs(a)))
            a * b = b * a
        }
    } else {
        if b.is_positive {
            b * a = -(Int.from_nat(abs(b) * abs(a)))
            a * b = b * a
        } else {
        }
    }
}

theorem mul_one_right(a: Int) { a * 1 = a } by {
    Nat.1 = Nat.0.suc
    a * Int.from_nat(Nat.0.suc) = a.mul_nat(Nat.0.suc)
    abs(a) * Nat.0.suc = abs(a)
    match a {
        Int.neg_suc(n) {
            -Int.from_nat(n.suc * Nat.0.suc) = Int.neg_suc(n).mul_nat(Nat.0.suc)
            -Int.from_nat(n.suc) = Int.neg_suc(n)
            Int.neg_suc(n).mul_nat(Nat.0.suc) = Int.neg_suc(n)
        }
        Int.from_nat(n) {
            Int.from_nat(n).mul_nat(Nat.0.suc) = Int.from_nat(n)
        }
    }
}

theorem mul_one_left(a: Int) { 1 * a = a }

theorem mul_neg_left(a: Int, b: Int) { -a * b = -(a * b) } by {
    match a {
        Int.from_nat(m) {
            match b {
                Int.from_nat(n) {
                    (-Int.from_nat(m)).mul_nat(n) = -Int.from_nat(m) * Int.from_nat(n)
                    Int.from_nat(m).mul_nat(n) = Int.from_nat(m) * Int.from_nat(n)
                    -a * b = -(a * b)
                }
                Int.neg_suc(n) {
                    Int.neg_suc(n).is_negative
                    -(-Int.from_nat(m)).mul_nat(n.suc) = -Int.from_nat(m) * Int.neg_suc(n)
                    -Int.from_nat(m).mul_nat(n.suc) = Int.from_nat(m) * Int.neg_suc(n)
                    -a * b = -(a * b)
                }
            }
        }
        Int.neg_suc(m) {
            match b {
                Int.from_nat(n) {
                    (-Int.neg_suc(m)).mul_nat(n) = -Int.neg_suc(m) * Int.from_nat(n)
                    Int.neg_suc(m).mul_nat(n) = Int.neg_suc(m) * Int.from_nat(n)
                    -a * b = -(a * b)
                }
                Int.neg_suc(n) {
                    Int.neg_suc(n).is_negative
                    -(-Int.neg_suc(m)).mul_nat(n.suc) = -Int.neg_suc(m) * Int.neg_suc(n)
                    -Int.neg_suc(m).mul_nat(n.suc) = Int.neg_suc(m) * Int.neg_suc(n)
                    -a * b = -(a * b)
                }
            }
        }
    }
}

theorem mul_neg_right(a: Int, b: Int) { a * -b = -(a * b) }

theorem mul_distrib_nonneg_right(a: Int, b: Int, c: Int) {
    not c.is_negative implies (a + b) * c = a * c + b * c
} by {
    (a + b) * c = a.mul_nat(abs(c)) + b.mul_nat(abs(c))
}

theorem mul_distrib_right(a: Int, b: Int, c: Int) { (a + b) * c = a * c + b * c } by {
    if c.is_negative {
        (a + b) * -(c) = -(a * c) + b * -(c)
    } else {
    }
}

theorem mul_distrib_left(a: Int, b: Int, c: Int) { a * (b + c) = a * b + a * c }

theorem mul_sub_distrib_right(a: Int, b: Int, c: Int) { (a - b) * c = a * c - b * c }

theorem mul_sub_distrib_left(a: Int, b: Int, c: Int) { a * (b - c) = a * b - a * c }

theorem abs_mul(a: Int, b: Int) { abs(a * b) = abs(a) * abs(b) } by {
    not a.is_positive or not a.is_negative
    not b.is_positive or not b.is_negative
    if a.is_positive {
        if b.is_positive {
            abs(a * b) = abs(a) * abs(b)
        } else {
            abs(a * b) = abs(a) * abs(b)
        }
    } else {
        if b.is_positive {
            abs(a * b) = abs(a) * abs(b)
        } else {
            abs(a * b) = abs(a) * abs(b)
        }
    }
}
