from lte import LTE
from nat import Nat
from nat import mod_maintains, mod_maintains_imp_gcd, pow_zero
from int.int_base import Int, abs, sub_nat

numerals Int

theorem mul_from_nat(j: Nat, k: Nat) { Int.from_nat(j) * Int.from_nat(k) = Int.from_nat(j * k) }

theorem abs_zero_imp_zero(a: Int) {
    abs(a) = Nat.0 implies a = 0
} by {
    if abs(a) = Nat.0 {
        Int.from_nat(Nat.0) = Int.0
        Int.from_nat(abs(a)) = Int.0
        -Int.0 = Int.0
        -Int.from_nat(abs(a)) = a or Int.from_nat(abs(a)) = a
        a = Int.0
    }
}

theorem mul_zero_imp_factor_zero(a: Int, b: Int) { a * b = 0 implies a = 0 or b = 0 } by {
    abs(a) * abs(b) = abs(a * b)
    abs(a) * abs(b) != Nat.0 or abs(a) = Nat.0 or abs(b) = Nat.0
    if abs(a) = Nat.0 {
    } else {
    }
}

theorem mul_pos_pos(a: Int, b: Int) {
    a.is_positive and b.is_positive implies (a * b).is_positive
} by {
    if a * b = 0 {
        if a = 0 {
        } else {
            false
        }
    } else {
        (a * b).is_positive
    }
}

theorem mul_pos_neg(a: Int, b: Int) {
    a.is_positive and b.is_negative implies (a * b).is_negative
}

theorem mul_neg_pos(a: Int, b: Int) {
    a.is_negative and b.is_positive implies (a * b).is_negative
}

theorem mul_neg_neg(a: Int, b: Int) {
    a.is_negative and b.is_negative implies (a * b).is_positive
}

theorem mul_int_nat_nat_assoc(a: Int, j: Nat, k: Nat) { a.mul_nat(j * k) = (a.mul_nat(j)).mul_nat(k) } by {
    if a.is_negative {
        // Simplify lhs
        a = -Int.from_nat(abs(a))
        a.mul_nat(j * k) = -Int.from_nat(abs(a) * (j * k))
        abs(a) * (j * k) = abs(a) * j * k

        // Simplify rhs
        a.mul_nat(j) = -Int.from_nat(abs(a) * j)
        (a.mul_nat(j)).mul_nat(k) = -Int.from_nat(abs(a) * j * k)

        // Conclusion
    } else {
        // Simplify lhs

        // Simplify rhs
        a.mul_nat(j) = Int.from_nat(abs(a) * j)
        (a.mul_nat(j)).mul_nat(k) = Int.from_nat(abs(a) * j * k)

        // Conclusion
        a.mul_nat(j * k) = (a.mul_nat(j)).mul_nat(k)
    }
}

theorem mul_int_int_nat_assoc(a: Int, b: Int, n: Nat) { (a * b).mul_nat(n) = a * (b).mul_nat(n) } by {
    if b.is_negative {
        b = -Int.from_nat(abs(b))
        a * b = -(a * Int.from_nat(abs(b)))
        (a * b).mul_nat(n) = -(a * Int.from_nat(abs(b) * n))
    } else {
        b.mul_nat(n) = Int.from_nat(abs(b) * n)
        (a * b).mul_nat(n) = a * Int.from_nat(abs(b) * n)
        (a * b).mul_nat(n) = a * b.mul_nat(n)
    }
}

theorem mul_assoc(a: Int, b: Int, c: Int) { a * b * c = a * (b * c) } by {
    if c.is_negative {
        a * b * c = -(a * b.mul_nat(abs(c)))
        a * b * c = a * (b * -Int.from_nat(abs(c)))
    } else {
        (a * b).mul_nat(abs(c)) = a * b.mul_nat(abs(c))
        a * b * c = a * (b * c)
    }
}

// Comparison operators

/// `a <= b` when `(a - b)` is positive or zero.
instance Int: LTE {
    define lte(self, b: Int) -> Bool {
        (b - self).is_positive or self = b
    }
}

attributes Int {
    /// True if this integer divides b (equivalently, there exists d such that d * this = b).
    define divides(self, b: Int) -> Bool {
        exists(d: Int) { d * self = b }
    }
}

from data.basic.relation_basic import is_reflexive, is_transitive, is_antisymmetric, transitive_step

theorem int_is_reflexive {
    is_reflexive(Int.lte)
}

theorem lte_trans(a: Int, b: Int, c: Int) {
    a <= b and b <= c implies a <= c
} by {
    if a = b {
    } else {
        if b = c {
            a <= c
        } else {
            (c - b).is_positive or c = b
            (b - a).is_positive or b = a
            (c - b).is_positive
            (b - a).is_positive
            ((c - b) + (b - a)).is_positive
            (c - b) + (b - a) = c - a
            a <= c
        }
    }
}

theorem int_is_transitive {
    is_transitive(Int.lte)
} by {
    if not is_transitive(Int.lte) {
        let (a: Int, b: Int, c: Int) satisfy {
            a <= b and b <= c and not (a <= c)
        }
        false
    }
}

theorem int_is_antisymmetric {
    is_antisymmetric(Int.lte)
} by {
    if not is_antisymmetric(Int.lte) {
        let (a: Int, b: Int) satisfy {
            a <= b and b <= a and a != b
        }
        (b - a).is_positive or a = b
        (a - b).is_positive or a = b
        not (b - a).is_positive or not (b - a).is_negative
        false
    }
}

from order import PartialOrder, LinearOrder
from order_relation import lt_relation_is_transitive

instance Int: PartialOrder

theorem lt_not_ref(a: Int) { not (a < a) }

theorem lte_ref(a: Int) { a <= a }

theorem zero_lt_pos(a: Int) { a.is_positive implies 0 < a } by {
    a.is_positive implies Int.0 <= a
}

theorem sub_nonnegative_of_lte(a: Int, b: Int) {
    a <= b implies Int.0 <= b - a
} by {
    if a <= b {
        a <= b = ((b - a).is_positive or a = b)
        if (b - a).is_positive {
            zero_lt_pos(b - a)
            Int.0 <= b - a
        } else {
            a = b
            Int.0 <= b - a
        }
    }
}

theorem from_nat_pos(n: Nat) {
    Nat.0 < n implies Int.from_nat(n).is_positive
} by {
    if Nat.0 < n {
        let k: Nat satisfy {
            k.suc = n
        }
        Int.from_nat(k.suc).is_positive
        Int.from_nat(n).is_positive
    }
}

theorem neg_lt_zero(a: Int) {
    a.is_negative implies a < 0
} by {
    a <= 0
}

theorem nonpos_lt_pos(a: Int, b: Int) {
    not a.is_positive and b.is_positive implies a < b
} by {
    (b - a).is_positive
}

theorem neg_lt_nonneg(a: Int, b: Int) { a.is_negative and not b.is_negative implies a < b }

theorem nonpos_lte_nonneg(a: Int, b: Int) {
    not a.is_positive and not b.is_negative implies a <= b
} by {
    b.is_positive or b.is_negative or b = Int.0
    a - Int.0 = a
    if not a <= b {
        a != b
        (a - b).is_positive or (b - a).is_positive
        (a - b).is_positive
        -(b - a) = a - b
        (-(b - a)).is_negative = (b - a).is_positive
        not b.is_positive
        b = Int.0
        false
    }
}

theorem lte_abs(a: Int) { a <= Int.from_nat(abs(a)) } by {
    if a.is_negative {
        not Int.from_nat(abs(a)).is_negative
        a <= Int.from_nat(abs(a))
    } else {
        a <= Int.from_nat(abs(a))
    }
}

theorem lt_add_left(a: Int, b: Int, c: Int) { b < c implies a + b < a + c } by {
    ((a + c) - (a + b)).is_positive
}

theorem lte_add_left(a: Int, b: Int, c: Int) { b <= c implies a + b <= a + c } by {
    not b <= c or b < c or c = b
    not b < c or a + b < a + c
    not a + b < a + c or a + b <= a + c
}

theorem lte_nonnegative_ints_implies_nats(n: Nat, m: Nat) {
    Int.from_nat(n) <= Int.from_nat(m) implies n <= m
} by {
    if Int.from_nat(n) = Int.from_nat(m) {
        n = m
    } else {
        Int.from_nat(m) - Int.from_nat(n) = sub_nat(m, n)
    }
}

theorem abs_add_nonneg(a: Int, b: Int) {
    not a.is_negative and not b.is_negative implies abs(a + b) = abs(a) + abs(b)
}

theorem abs_add_nonpos(a: Int, b: Int) {
    not a.is_positive and not b.is_positive implies abs(a + b) = abs(a) + abs(b)
}

// One case of the triangle inequality
theorem triangle_nonpos_lte_nonneg(a: Int, b: Int) {
    not a.is_negative and not b.is_positive and abs(b) <= abs(a) implies abs(a + b) <= abs(a) + abs(b)
} by {
    let (n: Nat) satisfy { n + abs(b) = abs(a) }
    Int.from_nat(n) + -b = a
    abs(a + b) = n
}

theorem triangle_nonneg_nonpos(a: Int, b: Int) {
    not a.is_negative and not b.is_positive implies abs(a + b) <= abs(a) + abs(b)
} by {
    if abs(b) <= abs(a) {
    } else {
        abs(-a) = abs(a)
        abs(-b) = abs(b)
        abs(-(a + b)) = abs(a + b)
        not (-b).is_negative
        not (-a).is_positive
        abs(-a) <= abs(-b)
        abs(-b + -a) <= abs(-b) + abs(-a)
        abs(a + b) <= abs(a) + abs(b)
    }
}

theorem triangle_ineq(a: Int, b: Int) { abs(a + b) <= abs(a) + abs(b) } by {
    if a.is_negative {
        not a.is_positive
        if b.is_negative {
            not b.is_positive
            abs(a + b) = abs(a) + abs(b)
            abs(a + b) <= abs(b) + abs(a)
        } else {
            not b.is_negative
            not a.is_positive
            abs(a) + abs(b) = abs(b) + abs(a)
            b + a = a + b
            abs(a + b) <= abs(b) + abs(a)
        }
    } else {
        not a.is_negative
        if b.is_negative {
            abs(a + b) <= abs(a) + abs(b)
        } else {
            not b.is_negative
            abs(a + b) = abs(a) + abs(b)
            abs(a + b) <= abs(b) + abs(a)
        }
    }
}

theorem lt_mul_pos(a: Int, b: Int, c: Int) { a < b and c.is_positive implies a * c < b * c } by {
    (b * c - a * c).is_positive
}

theorem lt_mul_neg(a: Int, b: Int, c: Int) { a < b and c.is_negative implies b * c < a * c } by {
    (a * c - b * c).is_positive
}

theorem lt_trans(a: Int, b: Int, c: Int) { a < b and b < c implies a < c } by {
    if a < b and b < c {
        lt_relation_is_transitive[Int]
        transitive_step(Int.lt, a, b, c)
        a < c
    }
}

theorem lt_and_lte(a: Int, b: Int, c: Int) { a < b and b <= c implies a < c } by {
    if b = c {
    } else {
    }
}

theorem lte_and_lt(a: Int, b: Int, c: Int) { a <= b and b < c implies a < c } by {
    if a = b {
    } else {
    }
}

theorem lt_from_nat(j: Nat, k: Nat) {
    j < k implies Int.from_nat(j) < Int.from_nat(k)
} by {
    let (d: Nat) satisfy { j + d = k and d != Nat.0 }
    Int.from_nat(d) + Int.from_nat(j) = Int.from_nat(k)
    Int.from_nat(d).is_positive
    Int.from_nat(j) <= Int.from_nat(k)
}

theorem lte_from_nat(j: Nat, k: Nat) { j <= k implies Int.from_nat(j) <= Int.from_nat(k) } by {
    not j <= k or j < k or k = j
    not j < k or Int.from_nat(j) < Int.from_nat(k)
    not Int.from_nat(j) < Int.from_nat(k) or Int.from_nat(j) <= Int.from_nat(k)
    if j = k {
    } else {
    }
}

// Units and dividing

define is_unit(a: Int) -> Bool { abs(a) = Nat.1 }

theorem two_units(u: Int) { is_unit(u) implies u = 1 or u = -(1) } by {
    if is_unit(u) {
        Int.from_nat(abs(u)) = Int.1
        if Int.from_nat(abs(u)) = u {
            u = Int.1
        }
        if -Int.from_nat(abs(u)) = u {
            u = -Int.1
        }
    }
}

theorem unit_squared(u: Int) { is_unit(u) implies u * u = 1 } by {
    if u = 1 {
    } else {
        u * u = 1
    }
}

theorem mul_units(u: Int, v: Int) { is_unit(u) and is_unit(v) implies is_unit(u * v) }

// Like the sign function, but we force it to be a unit, by considering 0 to have sign 1
define unit_sign(a: Int) -> Int {
    if a.is_negative {
        -(1)
    } else {
        1
    }
}

theorem unit_sign_is_unit(a: Int) { is_unit(unit_sign(a)) } by {
    a.is_negative implies unit_sign(a) = -Int.1
    not a.is_negative implies unit_sign(a) = Int.1
    unit_sign(a) = Int.from_nat(Nat.1) implies abs(unit_sign(a)) = Nat.1
    abs(unit_sign(a)) = Nat.1 implies is_unit(unit_sign(a))
    if a.is_negative {
        abs(-Int.1) = Nat.1
    } else {
        abs(Int.1) = Nat.1
    }
}

theorem abs_decomp(a: Int) { unit_sign(a) * Int.from_nat(abs(a)) = a } by {
    --Int.1 = Int.1
    Int.1 * Int.from_nat(abs(a)) = Int.from_nat(abs(a))
    not a.is_positive or not a.is_negative
    unit_sign(a) * Int.from_nat(abs(a)) = unit_sign(a).mul_nat(abs(a))
    sub_nat(a.pos_part, a.neg_part) = a
    --unit_sign(a) * Int.from_nat(abs(a)) = -(-unit_sign(a) * Int.from_nat(abs(a)))
    a + Int.0 = Int.0 + a
    sub_nat(a.pos_part, a.neg_part) + Int.0 = sub_nat(a.pos_part, a.neg_part)
    Int.0 + unit_sign(a).mul_nat(abs(a)) = unit_sign(a).mul_nat(abs(a))
    if a.is_negative {
        Int.from_nat(abs(a)).pos_part = abs(a)
    } else {
        unit_sign(a) * Int.from_nat(abs(a)) = a
    }
}

theorem abs_alt_decomp(a: Int) { unit_sign(a) * a = Int.from_nat(abs(a)) } by {
    unit_sign(a) * unit_sign(a) * Int.from_nat(abs(a)) = 1 * Int.from_nat(abs(a))
}

theorem div_trans(a: Int, b: Int, c: Int) {
    a.divides(b) and b.divides(c) implies a.divides(c)
} by {
    if a.divides(b) and b.divides(c) {
        let d1: Int satisfy { d1 * a = b }
        let d2: Int satisfy { d2 * b = c }
        d2 * d1 * a = c
        a.divides(c)
    }
    forall(d: Int, e: Int) {
        e * (d * a) = e * d * a
    }
}

theorem div_imp_div_abs(a: Int, b: Int) {
    a.divides(b) implies abs(a).divides(abs(b))
} by {
    let (d: Int) satisfy { d * a = b }
}

theorem div_from_nat(j: Nat, k: Nat) {
    j.divides(k) implies Int.from_nat(j).divides(Int.from_nat(k))
} by {
    let (n: Nat) satisfy { n * j = k }
}

theorem div_abs(a: Int) {
    a.divides(Int.from_nat(abs(a)))
}

theorem div_abs_imp_div(a: Int, b: Int) { abs(a).divides(abs(b)) implies a.divides(b) }

theorem abs_eq_imp_unit(a: Int, b: Int) {
    abs(a) = abs(b) implies exists(u: Int) { is_unit(u) and u * a = b }
} by {
    is_unit(unit_sign(b) * unit_sign(a))
}

theorem abs_eq_imp_div(a: Int, b: Int) { abs(a) = abs(b) implies a.divides(b) }

theorem div_pos_imp_lte(a: Int, b: Int) { a.divides(b) and b.is_positive implies a <= b } by {
    Int.from_nat(abs(a)) <= Int.from_nat(abs(b))
}

// Theorems that relate to the GCD

attributes Int {
    /// The greatest common divisor of this integer and b.
    define gcd(self, b: Int) -> Int {
        Int.from_nat(abs(self).gcd(abs(b)))
    }
}

theorem gcd_nonneg(a: Int, b: Int) {
    not a.gcd(b).is_negative
}

theorem gcd_comm(a: Int, b: Int) { a.gcd(b) = b.gcd(a) }

theorem gcd_div_left(a: Int, b: Int) { a.gcd(b).divides(a) } by {
    abs(a).gcd(abs(b)).divides(abs(a))
    abs(Int.from_nat(abs(a).gcd(abs(b)))) = abs(a).gcd(abs(b))
}

theorem gcd_div_right(a: Int, b: Int) { a.gcd(b).divides(b) } by {
}

theorem divides_gcd(a: Int, b: Int, d: Int) { d.divides(a) and d.divides(b) implies d.divides(a.gcd(b)) } by {
    d.divides(a) implies abs(d).divides(abs(a))
    d.divides(b) implies abs(d).divides(abs(b))
    abs(d).divides(abs(a)) and abs(d).divides(abs(b)) implies abs(d).divides(abs(a).gcd(abs(b)))
    abs(Int.from_nat(abs(a).gcd(abs(b)))) = abs(a).gcd(abs(b))
}

theorem gcd_pos(a: Int, b: Int) {
    a != 0 and b != 0 implies a.gcd(b).is_positive
} by {
    abs(a).gcd(abs(b)) != Nat.0
}

theorem gcd_is_gcd(a: Int, b: Int, d: Int) {
    a != 0 and b != 0 and d.divides(a) and d.divides(b) implies d <= a.gcd(b)
}

// Theorems about the span of a linear combination

define spans(a: Int, b: Int, c: Int) -> Bool {
    exists(d: Int, e: Int) {
        d * a + e * b = c
    }
}

theorem spans_zero(a: Int, b: Int) { spans(a, b, 0) } by {
    Int.0 * a + Int.0 * b = Int.0
}

theorem spans_left(a: Int, b: Int) { spans(a, b, a) } by {
    1 * a + 0 * b = a
}

theorem spans_comm(a: Int, b: Int, c: Int) { spans(a, b, c) implies spans(b, a, c) }

theorem spans_right(a: Int, b: Int) { spans(a, b, b) }

theorem spans_mul_left(a: Int, b: Int, c: Int) { spans(a, b, c * a) } by {
    c * a + Int.0 * b = c * a
}

theorem spans_div_left(a: Int, b: Int, c: Int) { a.divides(c) implies spans(a, b, c) }

theorem spans_mul_right(a: Int, b: Int, c: Int) { spans(a, b, c * b) }

theorem spans_div_right(a: Int, b: Int, c: Int) { b.divides(c) implies spans(a, b, c) }

theorem spans_mul(a: Int, b: Int, c: Int, d: Int) { spans(a, b, c) implies spans(a, b, d * c) } by {
    let (e: Int, f: Int) satisfy { e * a + f * b = c }
    d * c = d * e * a + d * f * b
}

theorem spans_add(a: Int, b: Int, c: Int, d: Int) { spans(a, b, c) and spans(a, b, d) implies spans(a, b, c + d) } by {
    let (e: Int, f: Int) satisfy { e * a + f * b = c }
    let (g: Int, h: Int) satisfy { g * a + h * b = d }
    c + d = (e * a + g * a) + (f * b + h * b)
}

theorem spans_negate(a: Int, b: Int, c: Int) { spans(a, b, c) implies spans(a, b, -(c)) }

let mod = Nat.mod

theorem spans_nat_mod(a: Int, b: Int, k: Nat, m: Nat) {
    spans(a, b, Int.from_nat(k)) and spans(a, b, Int.from_nat(m)) implies spans(a, b, Int.from_nat(mod(k, m)))
} by {
    let (d: Nat) satisfy { d * m + mod(k, m) = k }
    Int.from_nat(d * m) + Int.from_nat(mod(k, m)) = Int.from_nat(k)
    sub_nat(k, Nat.0) + -Int.from_nat(d * m) = sub_nat(k, Nat.0 + d * m)
    sub_nat(mod(k, m), Nat.0) = sub_nat(k, d * m)
    Int.from_nat(mod(k, m)) = Int.from_nat(k) + -(Int.from_nat(d) * Int.from_nat(m))
    spans(a, b, -Int.from_nat(d) * Int.from_nat(m))
    spans(a, b, Int.from_nat(k) + -(Int.from_nat(d) * Int.from_nat(m)))
}

// Bezout's identity
theorem spans_gcd(a: Int, b: Int) { spans(a, b, a.gcd(b)) } by {
    let f = function(n: Nat) {
        spans(a, b, Int.from_nat(n))
    }
    f(abs(a))
    // Prove mod_maintains(f) by showing the forall
    forall(n: Nat, m: Nat) {
        spans(a, b, Int.from_nat(m)) = f(m)
        f(n) and f(m) implies f(mod(n, m))
    }
    mod_maintains(f)
    f(abs(b))
    f(abs(a).gcd(abs(b)))
}

// More theorems analogous to nat theorems

theorem gcd_one_right(n: Int) {
    n.gcd(1) = 1
}

theorem gcd_one_left(n: Int) {
    1.gcd(n) = 1
}

theorem gcd_mult_left(a: Int, b: Int, m: Int) {
    Int.from_nat(abs(m)) * a.gcd(b) = (m * a).gcd(m * b)
}

theorem gcd_mult_right(a: Int, b: Int, m: Int) {
    a.gcd(b) * Int.from_nat(abs(m)) = (a * m).gcd(b * m)
}

theorem cofactor(a: Int, b: Int, af: Int, bf: Int) {
    (
        a.gcd(b) != 0 and
        af * a.gcd(b) = a and
        bf * a.gcd(b) = b
    ) implies af.gcd(bf) = 1
} by {
    abs(af) * abs(a).gcd(abs(b)) = abs(a)
    abs(bf) * abs(a).gcd(abs(b)) = abs(b)
    abs(af).gcd(abs(bf)) = Nat.1
}

theorem gcd_nonzero_left(a: Int, b: Int) {
    a != 0 implies a.gcd(b) != 0
} by {
    Int.from_nat(abs(a).gcd(abs(b))) = a.gcd(b)
    abs(a) != Nat.0 implies a != Int.0
    abs(b).gcd(abs(a)) = abs(a).gcd(abs(b))
    Nat.0.gcd(abs(a)) = abs(a)
    Int.from_nat(Nat.0).pos_part = Nat.0
    Int.from_nat(abs(a).gcd(abs(b))).pos_part = abs(a).gcd(abs(b))
}

theorem gcd_nonzero_right(a: Int, b: Int) {
    b != 0 implies a.gcd(b) != 0
}

theorem lt_mul_both(a: Int, b: Int, c: Int) {
    a.is_positive and b < c implies a * b < a * c
}

theorem lte_mul_both(a: Int, b: Int, c: Int) {
    a.is_positive and b <= c implies a * b <= a * c
}

theorem lt_mul_both_neg(a: Int, b: Int, c: Int) {
    a.is_negative and b < c implies a * b > a * c
}

theorem lte_mul_both_neg(a: Int, b: Int, c: Int) {
    a.is_negative and b <= c implies a * b >= a * c
} by {
    not b <= c or b < c or c = b
    not b < c or not a.is_negative or a * b > a * c
    a * b > a * c = a * c < a * b
    a * c < a * b implies a * c <= a * b
    a * c <= a * b = a * b >= a * c
    c = b implies a * b >= a * c
}

theorem lt_cancel_mul(a: Int, b: Int, c: Int) {
    a.is_positive and a * b < a * c implies b < c
} by {
    if a.is_positive and a * b < a * c {
        c <= b and a.is_positive implies a * c <= a * b
        a * c <= a * b and a * b < a * c implies a * b < a * b
        not a * b < a * b
        (c - b).is_positive implies b <= c
        -(c - b) = b - c
        b - c != Int.0 or c = b
        (b - c).is_positive or (b - c).is_negative or b - c = Int.0
        (-(c - b)).is_negative = (c - b).is_positive
        b <= c
    }
}

theorem lte_cancel_mul(a: Int, b: Int, c: Int) {
    a.is_positive and a * b <= a * c implies b <= c
} by {
    c <= b implies c < b or c = b
    (c - b).is_positive implies b <= c
    -(c - b) = b - c
    b - c != Int.0 or c = b
    (b - c).is_positive or (b - c).is_negative or b - c = Int.0
    (-(c - b)).is_negative = (c - b).is_positive
    if not b <= c {
        not c < b
    }
}

theorem lt_cancel_mul_neg(a: Int, b: Int, c: Int) {
    a.is_negative and a * b < a * c implies b > c
} by {
    a.is_negative and a * b < a * c implies a * (a * b) > a * (a * c)
    a * (a * b) = a * a * b
    a * (a * c) = a * a * c
    a.is_negative implies (a * a).is_positive
    b > c = c < b
}

theorem lte_cancel_mul_neg(a: Int, b: Int, c: Int) {
    a.is_negative and a * b <= a * c implies b >= c
} by {
    a * b <= a * c and a.is_negative implies a * (a * b) >= a * (a * c)
    a * (a * b) = a * a * b
    a * (a * c) = a * a * c
    a.is_negative implies (a * a).is_positive
    c <= b = b >= c
}

theorem mul_pos_cancel_left(a: Int, b: Int, c: Int) {
    a.is_positive and a * b = a * c implies b = c
} by {
    if b != c {
        if b < c {
        } else {
            false
        }
    }
}

theorem mul_pos_cancel_right(a: Int, b: Int, c: Int) {
    a.is_positive and b * a = c * a implies b = c
}

theorem mul_neg_cancel_left(a: Int, b: Int, c: Int) {
    a.is_negative and a * b = a * c implies b = c
} by {
    (-a) * b = (-a) * c
}

theorem mul_neg_cancel_right(a: Int, b: Int, c: Int) {
    a.is_negative and b * a = c * a implies b = c
}

theorem mul_cancel_left(a: Int, b: Int, c: Int) {
    a != 0 and a * b = a * c implies b = c
} by {
    if a.is_positive {
    } else {
    }
}

theorem mul_cancel_right(a: Int, b: Int, c: Int) {
    a != 0 and b * a = c * a implies b = c
}

theorem divides_cancel_left(a: Int, b: Int, c: Int) {
    a != 0 and (a * b).divides(a * c) implies b.divides(c)
} by {
    if (a * b).divides(a * c) {
        let d: Int satisfy { d * (a * b) = a * c }
        a * (d * b) = a * d * b
        a * (d * b) = a * c
        d * b = c
    }
}

theorem divides_cancel_right(a: Int, b: Int, c: Int) {
    a != 0 and (b * a).divides(c * a) implies b.divides(c)
}

theorem divides_mul_left(a: Int, b: Int, m: Int) {
    a.divides(b) implies (m * a).divides(m * b)
} by {
    let d: Int satisfy {
        d * a = b
    }
    d * (m * a) = m * d * a
    d * (m * a) = m * b
}

theorem divides_mul_right(a: Int, b: Int, m: Int) {
    a.divides(b) implies (a * m).divides(b * m)
}

define is_prime(a: Int) -> Bool {
    abs(a).is_prime
}

theorem gcd_of_prime(p: Int, n: Int) {
    is_prime(p) implies p.gcd(n) = 1 or p.divides(n)
} by {
    abs(p).is_prime = is_prime(p)
    abs(p).is_prime implies abs(p).gcd(abs(n)) = Nat.1 or abs(p).divides(abs(n))
    abs(p).divides(abs(n)) implies p.divides(n)
    if abs(p).gcd(abs(n)) = Nat.1 {
    } else {
    }
}

// Generalized version
theorem euclids_lemma(a: Int, b: Int, c: Int) {
    a.gcd(b) = 1 and a.divides(b * c) implies a.divides(c)
} by {
    spans(a, b, 1)
    let (x: Int, y: Int) satisfy {
        x * a + y * b = 1
    }
    let d: Int satisfy {
        d * a = b * c
    }
    c * (x * a) = c * x * a
    c * (x * a) = x * a * c
    c * (y * b) = y * b * c
    c * (x * a + y * b) = c * x * a + c * y * b
    c * x * a + y * d * a = (c * x + y * d) * a
    (c * x + y * d) * a != c or a.divides(c)
    x * a * c + y * d * a = c
}

theorem euclids_lemma_prime(a: Int, b: Int, c: Int) {
    is_prime(a) and a.divides(b * c) implies a.divides(b) or a.divides(c)
} by {
    if a.divides(b) {
    } else {
    }
}

theorem one_plus_one {
    1 + 1 = 2
}

theorem times_two(a: Int) {
    2 * a = a + a
}

// This is just the Nat division theorem ported up
theorem positive_division_theorem(m: Int, n: Int) {
    0 <= m and n.is_positive implies exists(q: Int, r: Int) {
        0 <= r and r < n and m = q * n + r
    }
} by {
    let (q: Nat, r: Nat) satisfy {
        r < abs(n) and abs(m) = q * abs(n) + r
    }
    0 <= Int.from_nat(r)
    n.pos_part = abs(n)
    n.neg_part = Nat.0
    sub_nat(abs(n), Nat.0) = Int.from_nat(abs(n))
    sub_nat(n.pos_part, n.neg_part) = n
    n = Int.from_nat(abs(n))
    Int.from_nat(r) < Int.from_nat(abs(n))
    m = Int.from_nat(q) * Int.from_nat(abs(n)) + Int.from_nat(r)
}

// Dividing negative numbers is awkward, sometimes you're off by one
theorem negative_division_theorem(m: Int, n: Int) {
    m.is_negative and n.is_positive implies exists(q: Int, r: Int) {
        0 <= r and r < n and m = q * n + r
    }
} by {
    0 <= -m
    let (qn: Int, rn: Int) satisfy {
        0 <= rn and rn < n and -m = qn * n + rn
    }
    m = (-qn) * n + -rn
    if rn = 0 {
        exists(q: Int, r: Int) {
            0 <= r and r < n and m = q * n + r
        }
    } else {
        (-qn + -1) * n = (-qn) * n + (-1) * n
        (-1) * n = -n
        (-qn + -1) * n + n = (-qn) * n + -n + n
        -n + n = Int.0
        (-qn) * n = (-qn + -1) * n + n
        m = (-qn + -1) * n + (n + -rn)
        0 <= n + -rn
        -rn < 0
        n + -rn < n
        exists(q: Int, r: Int) {
            0 <= r and r < n and m = q * n + r
        }
    }
}

// The entire division theorem is combining the two cases
theorem division_theorem(m: Int, n: Int) {
    n.is_positive implies exists(q: Int, r: Int) {
        0 <= r and r < n and m = q * n + r
    }
} by {
    if m.is_negative {
        exists(q: Int, r: Int) {
            0 <= r and r < n and m = q * n + r
        }
    } else {
        0 <= m
        exists(q: Int, r: Int) {
            0 <= r and r < n and m = q * n + r
        }
    }
}

theorem add_lte(a: Int, b: Int, c: Int, d: Int) {
    a <= b and c <= d implies a + c <= b + d
} by {
    a <= b implies c + a <= c + b
    c <= d implies b + c <= b + d
    c + a <= c + b and c + b <= b + d implies c + a <= b + d
    c + a = a + c
    c + b = b + c
}

theorem plus_abs_gte_zero(n: Int) {
    n + Int.from_nat(abs(n)) >= 0
} by {
    -n <= Int.from_nat(abs(n))
    n + -n <= n + Int.from_nat(abs(n))
}

from algebra.semigroup import Semigroup

instance Int: Semigroup

from algebra.monoid.monoid import Monoid

instance Int: Monoid

theorem exp_one(a: Int) {
    a.pow(Nat.1) = a
}

theorem exp_zero(a: Int) {
    a.pow(Nat.0) = 1
}

theorem exp_add(a: Int, b: Nat, c: Nat) {
    a.pow(b + c) = a.pow(b) * a.pow(c)
} by {
    // Inductive step
    let f = function(x: Nat) {
        a.pow(b + x) = a.pow(b) * a.pow(x)
    }
}

theorem exp_mul(a: Int, b: Nat, c: Nat) {
    a.pow(b * c) = a.pow(b).pow(c)
} by {
    // Inductive step
    let f = function(x: Nat) {
        a.pow(b * x) = a.pow(b).pow(x)
    }
}

theorem zero_exp(n: Nat) {
    n != Nat.0 implies 0.pow(n) = 0
} by {
    let f = function(k: Nat) {
        k != Nat.0 implies Int.0.pow(k) = Int.0
    }
    f(Nat.0)
    forall(m: Nat) {
        Int.0 * Int.0.pow(m) = Int.0
        f(m.suc)
    }
    forall(k: Nat) { f(k) }
}

theorem one_exp(n: Nat) {
    1.pow(n) = 1
}

theorem pos_exp(a: Int, n: Nat) {
    a.is_positive implies a.pow(n).is_positive
} by {
    let f = function(x: Nat) {
        a.pow(x).is_positive
    }
    f(Nat.0)
    forall(k: Nat) {
        a * a.pow(k) = a.pow(k.suc)
        if f(k) {
            a.is_positive and a.pow(k).is_positive implies (a * a.pow(k)).is_positive
            f(k.suc)
        }
    }
}

theorem sq_eq_mul(a: Int) {
    a * a = a.pow(Nat.2)
}

theorem sq_pos(a: Int) {
    a != 0 implies a.pow(Nat.2).is_positive
}

theorem exp_abs_eq_abs_exp(a: Int, n: Nat) {
    abs(a.pow(n)) = abs(a).pow(n)
} by {
    let f = function(x: Nat) {
        abs(a.pow(x)) = abs(a).pow(x)
    }
    f(Nat.0)
    forall(x: Nat) {
        if f(x) {
            abs(a.pow(x.suc)) = abs(a).pow(x.suc)
        }
    }
}

attributes Int {
    /// The absolute value of an integer.
    define abs(self) -> Int {
        Int.from_nat(abs(self))
    }
}

theorem member_abs_neg(a: Int) {
    a.is_negative implies a.abs = -a
}

theorem member_abs_nonneg(a: Int) {
    not a.is_negative implies a.abs = a
}

theorem member_abs_pos(a: Int) {
    a.is_positive implies a.abs = a
}

theorem member_abs_nonpos(a: Int) {
    not a.is_positive implies a.abs = -a
}

theorem member_triangle(a: Int, b: Int) {
    (a + b).abs <= a.abs + b.abs
} by {
    Int.from_nat(abs(a)) + Int.from_nat(abs(b)) = Int.from_nat(abs(a) + abs(b))
    Int.from_nat(abs(a + b)) <= Int.from_nat(abs(a) + abs(b))
}

// Connecting Int to its multiplicative algebraic structure.

from semiring import Semiring

instance Int: Semiring

from algebra.ring.ring import Ring

instance Int: Ring

from algebra.comm_semigroup import CommSemigroup

instance Int: CommSemigroup

from algebra.comm_monoid import CommMonoid

instance Int: CommMonoid

from comm_ring import CommRing

instance Int: CommRing

theorem int_total(a: Int, b: Int) { a <= b or b <= a } by {
    if a = b {
        a <= b
    } else {
        a - b != Int.0
        if a - b = 0 {
        } else {
            if (a - b).is_positive {
                b <= a
            } else {
                (a - b).is_negative
                (-(b - a)).is_negative = (b - a).is_positive
                a <= b
            }
        }
    }
}

instance Int: LinearOrder

from algebra.add_ordered_group import AddLeftOrderedGroup, AddOrderedGroup, add_lt_add_left,
    add_lt_add_right

instance Int: AddLeftOrderedGroup
instance Int: AddOrderedGroup

theorem add_sub_lt_right_of_lt(a: Int, b: Int, c: Int) {
    a < b implies a + c - b < c
} by {
    if a < b {
        add_lt_add_right[Int](a, b, c)
        add_lt_add_left[Int](-b, a + c, b + c)
        -b + (a + c) < -b + (b + c)
        -b + b = Int.0
        a + c - b < c
    }
}

from lattice import Meet, Join, MeetSemilattice, JoinSemilattice, Lattice, DistribLattice
from order import min_lte_left, min_lte_right, lte_min_of_bounds, lte_max_left, lte_max_right,
    max_lte_of_upper_bounds, min_max_distrib_left, max_min_distrib_left

instance Int: Meet {
    define meet(self, other: Int) -> Int {
        self.min(other)
    }
}

instance Int: Join {
    define join(self, other: Int) -> Int {
        self.max(other)
    }
}

theorem int_meet_lte_left(a: Int, b: Int) {
    a.meet(b) <= a
} by {
    min_lte_left(a, b)
}

theorem int_meet_lte_right(a: Int, b: Int) {
    a.meet(b) <= b
} by {
    min_lte_right(a, b)
}

theorem int_lte_meet_of_bounds(c: Int, a: Int, b: Int) {
    c <= a and c <= b implies c <= a.meet(b)
} by {
    if c <= a and c <= b {
        lte_min_of_bounds(c, a, b)
        c <= a.meet(b)
    }
}

instance Int: MeetSemilattice

theorem int_lte_join_left(a: Int, b: Int) {
    a <= a.join(b)
} by {
    lte_max_left(a, b)
}

theorem int_lte_join_right(a: Int, b: Int) {
    b <= a.join(b)
} by {
    lte_max_right(a, b)
}

theorem int_join_lte_of_bounds(a: Int, b: Int, c: Int) {
    a <= c and b <= c implies a.join(b) <= c
} by {
    if a <= c and b <= c {
        max_lte_of_upper_bounds(a, b, c)
        a.join(b) <= c
    }
}

instance Int: JoinSemilattice

instance Int: Lattice

theorem int_meet_join_distrib_left(a: Int, b: Int, c: Int) {
    a.meet(b.join(c)) = a.meet(b).join(a.meet(c))
} by {
    min_max_distrib_left(a, b, c)
}

theorem int_join_meet_distrib_left(a: Int, b: Int, c: Int) {
    a.join(b.meet(c)) = a.join(b).meet(a.join(c))
} by {
    max_min_distrib_left(a, b, c)
}

instance Int: DistribLattice
