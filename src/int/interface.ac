/// Public interface for integers.

from algebra.add import Add
from algebra.mul import Mul
from nat import Nat
from algebra.neg import Neg
from algebra.zero import Zero
from algebra.one import One
from algebra.add_semigroup import AddSemigroup
from algebra.add_comm_semigroup import AddCommSemigroup
from algebra.add_monoid import AddMonoid
from algebra.add_comm_monoid import AddCommMonoid
from algebra.add_group import AddGroup
from algebra.add_comm_group import AddCommGroup
from lte import LTE
from nat import mod_maintains
from data.basic.relation_basic import is_reflexive, is_transitive, is_antisymmetric, transitive_step
from order import PartialOrder, LinearOrder, min_lte_left, min_lte_right, lte_min_of_bounds,
    lte_max_left, lte_max_right, max_lte_of_upper_bounds, min_max_distrib_left,
    max_min_distrib_left
from order_relation import lt_relation_is_transitive
from algebra.semigroup import Semigroup
from algebra.monoid.monoid import Monoid
from semiring import Semiring
from algebra.ring.ring import Ring
from algebra.comm_semigroup import CommSemigroup
from algebra.comm_monoid import CommMonoid
from comm_ring import CommRing
from algebra.add_ordered_group import AddLeftOrderedGroup, AddOrderedGroup
from lattice import Meet, Join, MeetSemilattice, JoinSemilattice, Lattice, DistribLattice

// int_base.ac

/// The `Int` type represents integers.
/// It's defined by its two constructors.
///
/// `from_nat` takes a natural number to an integer, which seems intuitive.
/// `neg_suc` takes `x` to `-(x+1)`, which is somewhat less intuitive. We do this so
/// that every integer can be represented either as a `from_nat` or a `neg_suc`.
///
/// ```acorn
/// numerals Int
///
/// 2 = Int.from_nat(Nat.2)
/// -2 = Int.neg_suc(Nat.1)
/// ```
inductive Int {
    /// `Int.from_nat` converts a natural number to an integer via the typical embedding.
    from_nat(Nat)

    /// `Int.neg_suc` converts a natural number `x` into `-(x+1)`.
    /// This isn't particularly intuitive, it's just to give every integer a unique constructor.
    /// In particular, `neg_suc` can construct any negative integer, but not zero.
    neg_suc(Nat)
}

let from_nat = Int.from_nat

/// Yields the absolute value of an integer as a natural number.
define abs(a: Int) -> Nat {
    match a {
        Int.from_nat(n) {
            n
        }
        Int.neg_suc(k) {
            k.suc
        }
    }
}

theorem abs_from_nat(n: Nat) {
    abs(Int.from_nat(n)) = n
}

attributes Int {
    /// The integer two.
    let 2: Int = Int.from_nat(Nat.2)
    /// The integer three.
    let 3: Int = Int.from_nat(Nat.3)
    /// The integer four.
    let 4: Int = Int.from_nat(Nat.4)
    /// The integer five.
    let 5: Int = Int.from_nat(Nat.5)
    /// The integer six.
    let 6: Int = Int.from_nat(Nat.6)
    /// The integer seven.
    let 7: Int = Int.from_nat(Nat.7)
    /// The integer eight.
    let 8: Int = Int.from_nat(Nat.8)
    /// The integer nine.
    let 9: Int = Int.from_nat(Nat.9)
    /// The integer ten.
    let 10: Int = Int.from_nat(Nat.10)
}


instance Int: Zero {
    let 0: Int = Int.from_nat(Nat.0)
}


instance Int: One {
    let 1: Int = Int.from_nat(Nat.1)
}

numerals Int

/// Converts a natural number to its negative integer equivalent.
define neg_nat(n: Nat) -> Int {
    if n = Nat.0 {
        0
    } else {
        Int.neg_suc(n - Nat.1)
    }
}

theorem neg_nat_zero { neg_nat(Nat.0) = 0 }

theorem neg_nat_suc(n: Nat) {
    neg_nat(n.suc) = Int.neg_suc(n)
}

theorem abs_neg_nat(n: Nat) {
    abs(neg_nat(n)) = n
}

/// The negation of an integer.
instance Int: Neg {
    define neg(self) -> Int {
        match self {
            Int.from_nat(n) {
                neg_nat(n)
            }
            Int.neg_suc(n) {
                Int.from_nat(n.suc)
            }
        }
    }
}

theorem neg_zero {
    -0 = 0
}

theorem neg_neg_suc(n: Nat) {
    -Int.neg_suc(n) = Int.from_nat(n.suc)
}

theorem neg_from_nat(n: Nat) {
    -Int.from_nat(n) = neg_nat(n)
}

theorem neg_neg(a: Int) {
    -(-a) = a
}

theorem fix_neg(a: Int) {
    -a = a implies a = 0
}

theorem abs_neg(a: Int) {
    abs(-a) = abs(a)
}

theorem neg_or_pos(a: Int) {
    a = Int.from_nat(abs(a)) or a = -(Int.from_nat(abs(a)))
}

theorem from_eq_neg_from(p: Nat, q: Nat) {
    Int.from_nat(p) = -(Int.from_nat(q)) implies p = Nat.0 and q = Nat.0
}

theorem abs_zero { abs(0) = Nat.0 }

theorem one_neq_zero { 1 != 0 }

// Subtraction that goes from naturals into integers.
// We will use this as the primary representation for proving things about integers, so we prove
// as many useful things about sub_nat as we can, before defining more stuff.
define sub_nat(m: Nat, n: Nat) -> Int {
    if n <= m {
        Int.from_nat(m - n)
    } else {
        -(Int.from_nat(n - m))
    }
}

theorem sub_nat_zero_right(n: Nat) {
    sub_nat(n, Nat.0) = Int.from_nat(n)
}

theorem sub_nat_zero_left(n: Nat) { sub_nat(Nat.0, n) = -(Int.from_nat(n)) }

theorem sub_nat_self(n: Nat) { sub_nat(n, n) = 0 }

theorem sub_nat_add_left(p: Nat, q: Nat) {
    sub_nat(p + q, q) = Int.from_nat(p)
}

theorem neg_sub_nat(m: Nat, n: Nat) { sub_nat(m, n) = -(sub_nat(n, m)) }

theorem sub_nat_add_right(p: Nat, q: Nat) { sub_nat(p, p + q) = -(Int.from_nat(q)) }

// Half of a "without loss of generality" argument
theorem sub_nat_eq_helper(m: Nat, n: Nat, p: Nat, q: Nat) {
    m + n = p + q and p <= m implies sub_nat(m, p) = sub_nat(q, n)
}

theorem sub_nat_eq(m: Nat, n: Nat, p: Nat, q: Nat) {
    m + n = p + q implies sub_nat(m, p) = sub_nat(q, n)
}

theorem sub_nat_imp_add(i: Nat, j: Nat, k: Nat) {
    sub_nat(i, j) = Int.from_nat(k) implies j + k = i
}

theorem sub_nat_negate_imp_add(i: Nat, j: Nat, k: Nat) {
    sub_nat(i, j) = -(Int.from_nat(k)) implies i + k = j
}

theorem sub_nat_cancel_right(i: Nat, j: Nat, k: Nat) { sub_nat(i, k) = sub_nat(j, k) implies i = j }

theorem sub_nat_cancel_left(i: Nat, j: Nat, k: Nat) {
    sub_nat(k, i) = sub_nat(k, j) implies i = j
}

theorem sub_nat_add_cancel_right(m: Nat, n: Nat, k: Nat) {
    sub_nat(m, n) = sub_nat(m + k, n + k)
}

theorem sub_nat_add_cancel_left(m: Nat, n: Nat, k: Nat) {
    sub_nat(m, n) = sub_nat(k + m, k + n)
}

// Half of a "without loss of generality" argument
theorem sub_nat_imp_add_eq_helper(m: Nat, n: Nat, p: Nat, q: Nat) {
    sub_nat(m, p) = sub_nat(q, n) and p <= m implies m + n = p + q
}

theorem sub_nat_imp_add_eq(m: Nat, n: Nat, p: Nat, q: Nat) { sub_nat(m, p) = sub_nat(q, n) implies m + n = p + q }

theorem sub_nat_double_cancel_left(p: Nat, q: Nat, r: Nat, s: Nat, t: Nat) {
    sub_nat(p + t, q) = sub_nat(r + t, s) implies sub_nat(p, q) = sub_nat(r, s)
}

theorem sub_nat_double_cancel_right(p: Nat, q: Nat, r: Nat, s: Nat, t: Nat) {
    sub_nat(p, q + t) = sub_nat(r, s + t) implies sub_nat(p, q) = sub_nat(r, s)
}

// Now that we've proven a bunch of stuff about sub_nat, we define the positive and negative parts so that we can
// represent each integer as a sub_nat, and start defining useful functions on integers.

attributes Int {
    /// True if the integer is negative.
    define is_negative(self) -> Bool {
        self != Int.from_nat(abs(self))
    }

    /// True if the integer is positive.
    define is_positive(self) -> Bool {
        (-self).is_negative
    }
}

theorem zero_not_neg { not 0.is_negative }

theorem zero_not_pos {
    not 0.is_positive
}

theorem one_pos { 1.is_positive }

theorem nonzero_pos_or_neg(a: Int) {
    a != 0 implies a.is_positive or a.is_negative
}

theorem pos_is_not_neg(a: Int) { a.is_positive implies not a.is_negative }

theorem non_pos_is_neg_abs(a: Int) { not a.is_positive implies a = -(Int.from_nat(abs(a))) }

attributes Int {
    /// The positive part of this integer.
    define pos_part(self) -> Nat {
        if self.is_positive {
            abs(self)
        } else {
            Nat.0
        }
    }

    /// The negative part of this integer.
    define neg_part(self) -> Nat {
        if self.is_positive {
            Nat.0
        } else {
            abs(self)
        }
    }
}

theorem sub_nat_parts(a: Int) { sub_nat(a.pos_part, a.neg_part) = a }

theorem pos_part_neg(a: Int) { (-a).pos_part = a.neg_part }

theorem pos_part_from(n: Nat) { Int.from_nat(n).pos_part = n }

theorem neg_part_from(n: Nat) {
    Int.from_nat(n).neg_part = Nat.0
}

theorem neg_part_neg(a: Int) { (-a).neg_part = a.pos_part }

theorem parts_sub_nat(j: Nat, k: Nat) { sub_nat(j, k).pos_part + k = sub_nat(j, k).neg_part + j }

theorem add_part_sub_nat(r: Nat, s: Nat) { r + sub_nat(r, s).neg_part = s + sub_nat(r, s).pos_part }

// Addition, and theorems about addition

/// The sum of two integers.
instance Int: Add {
    define add(self, other: Int) -> Int {
        sub_nat(self.pos_part + other.pos_part, self.neg_part + other.neg_part)
    }
}

theorem add_zero_left(a: Int) { (0 + a) = a }

theorem add_zero_right(a: Int) { a + 0 = a }

theorem add_comm(a: Int, b: Int) { a + b = b + a }

theorem neg_distrib(a: Int, b: Int) { -(a + b) = -a + -b }

theorem parts_of_add(a: Int, b: Int) {
    (a.pos_part + b.pos_part + (a + b).neg_part =
     a.neg_part + b.neg_part + (a + b).pos_part)
}

theorem add_neg(a: Int) { a + -a = 0 }

theorem add_eq_zero(a: Int, b: Int) { a + b = 0 implies a = -b }

theorem add_right_cancel(a: Int, b: Int, c: Int) { a + c = b + c implies a = b }

theorem add_left_cancel(a: Int, b: Int, c: Int) { c + a = c + b implies a = b }

theorem add_sub_nat_left_pos(p: Nat, q: Nat, r: Nat) {
    (sub_nat(p, q) + Int.from_nat(r)) = sub_nat(p + r, q)
}

theorem add_sub_nat_left_neg(p: Nat, q: Nat, r: Nat) {
    (sub_nat(p, q) + -(Int.from_nat(r))) = sub_nat(p, q + r)
}

theorem add_sub_nat_left(p: Nat, q: Nat, a: Int) {
    (sub_nat(p, q) + a) = sub_nat(p + a.pos_part, q + a.neg_part)
}

theorem add_sub_nat_right(p: Nat, q: Nat, a: Int) {
    a + sub_nat(p, q) = sub_nat(a.pos_part + p, a.neg_part + q)
}

theorem add_sub_nat(p: Nat, q: Nat, r: Nat, s: Nat) {
    (sub_nat(p, q) + sub_nat(r, s)) = sub_nat(p + r, q + s)
}

theorem add_sub_nat_3_left(p: Nat, q: Nat, r: Nat, s: Nat, t: Nat, u: Nat) {
    sub_nat(p, q) + sub_nat(r, s) + sub_nat(t, u) = sub_nat(p + r + t, q + s + u)
}

theorem add_sub_nat_3_right(p: Nat, q: Nat, r: Nat, s: Nat, t: Nat, u: Nat) {
    sub_nat(p, q) + (sub_nat(r, s) + sub_nat(t, u)) = sub_nat(p + (r + t), q + (s + u))
}

theorem add_sub_nat_assoc(p: Nat, q: Nat, r: Nat, s: Nat, t: Nat, u: Nat) {
    sub_nat(p, q) + sub_nat(r, s) + sub_nat(t, u) = sub_nat(p, q) + (sub_nat(r, s) + sub_nat(t, u))
}

theorem add_assoc(a: Int, b: Int, c: Int) { a + (b + c) = a + b + (c) }

theorem add_from_nat(a: Nat, b: Nat) {
    Int.from_nat(a) + Int.from_nat(b) = Int.from_nat(a + b)
}

theorem add_pos_nonneg(a: Int, b: Int) {
    a.is_positive and not b.is_negative implies (a + b).is_positive
}

theorem add_neg_nonpos(a: Int, b: Int) {
    a.is_negative and not b.is_positive implies (a + b).is_negative
}

theorem add_nonneg_nonneg(a: Int, b: Int) {
    not a.is_negative and not b.is_negative implies not (a + b).is_negative
}

theorem add_nonpos_nonpos(a: Int, b: Int) {
    not a.is_positive and not b.is_positive implies not (a + b).is_positive
}

theorem add_comm_4(a: Int, b: Int, c: Int, d: Int) { (a + b) + (c + d) = (a + c) + (b + d) }

// Connecting Int to its additive algebraic structure.


instance Int: AddSemigroup


instance Int: AddCommSemigroup


instance Int: AddMonoid


instance Int: AddCommMonoid


instance Int: AddGroup


instance Int: AddCommGroup

// Subtraction, and theorems about subtraction

theorem sub_zero_right(a: Int) { a - 0 = a }

theorem sub_zero_left(a: Int) { 0 - a = -a }

theorem sub_anticomm(a: Int, b: Int) { a - b = -(b - a) }

theorem sub_self(a: Int) { a - a = 0 }

theorem sub_eq_zero(a: Int, b: Int) { a - b = 0 implies a = b }

theorem sub_add_left(a: Int, b: Int) {
    (a + b) - b = a
}

theorem neg_sub(a: Int, b: Int) { -(a - b) = b - a }

theorem sub_add_right(a: Int, b: Int) { a - (a + b) = -b }

theorem sub_imp_add(a: Int, b: Int, c: Int) { a - b = c implies b + c = a }

theorem sub_negate_imp_add(a: Int, b: Int, c: Int) { a - b = -(c) implies a + c = b }

theorem sub_cancel_right(a: Int, b: Int, c: Int) { a - c = b - c implies a = b }

theorem sub_cancel_left(a: Int, b: Int, c: Int) { a - b = a - c implies b = c }

theorem sub_add_cancel_left(a: Int, b: Int, c: Int) { (a + b) - (a + c) = b - c }

theorem sub_add_cancel_right(a: Int, b: Int, c: Int) { (a + c) - (b + c) = a - b }

// Integer-natural multiplication

attributes Int {
    /// Multiply this integer by a natural number.
    define mul_nat(self, n: Nat) -> Int {
        if self.is_negative {
            -(Int.from_nat(abs(self) * n))
        } else {
            Int.from_nat(abs(self) * n)
        }
    }
}

theorem mul_nat_zero_right(a: Int) { a.mul_nat(Nat.0) = Int.0 }

theorem mul_nat_zero_left(n: Nat) { 0.mul_nat(n) = 0 }

theorem mul_nat_nonpos_left(a: Int, n: Nat) {
    not a.is_positive implies a.mul_nat(n) = -(Int.from_nat(abs(a) * n))
}

theorem mul_nat_negate_left(a: Int, n: Nat) { (-a).mul_nat(n) = -(a.mul_nat(n)) }

theorem mul_nat_nonneg_suc(a: Int, n: Nat) {
    not a.is_negative implies a.mul_nat(n.suc) = a.mul_nat(n) + a
}

theorem mul_nat_suc(a: Int, n: Nat) {
    a.mul_nat(n.suc) = (a.mul_nat(n) + a)
}

theorem mul_nat_distrib_right(a: Int, b: Int, n: Nat) {
    (a + b).mul_nat(n) = (a.mul_nat(n) + b.mul_nat(n))
}

theorem mul_nat_from_nat_left(a: Nat, b: Nat) { Int.from_nat(a).mul_nat(b) = Int.from_nat(a * b) }

// Integer-integer multiplication

/// The product of two integers.
instance Int: Mul {
    define mul(self, n: Int) -> Int {
        if n.is_positive {
            self.mul_nat(abs(n))
        } else {
            -(self.mul_nat(abs(n)))
        }
    }
}

attributes Int {
    /// The integer formed by appending a digit to this integer in base 10.
    define read(self, other: Int) -> Int { 10 * self + other }
}

theorem mul_zero_right(a: Int) { a * 0 = 0 }

theorem mul_nat_from_nat_right(a: Int, n: Nat) { a.mul_nat(n) = (a * Int.from_nat(n)) }

theorem mul_nonneg_right(a: Int, b: Int) { not b.is_negative implies a * b = a.mul_nat(abs(b)) }

theorem mul_nonneg_nonneg(a: Int, b: Int) {
    not a.is_negative and not b.is_negative implies a * b = Int.from_nat(abs(a) * abs(b))
}

theorem mul_nonneg_nonpos(a: Int, b: Int) {
    not a.is_negative and not b.is_positive implies a * b = -(Int.from_nat(abs(a) * abs(b)))
}

theorem mul_nonpos_nonneg(a: Int, b: Int) {
    not a.is_positive and not b.is_negative implies a * b = -(Int.from_nat(abs(a) * abs(b)))
}

theorem mul_nonpos_nonpos(a: Int, b: Int) {
    not a.is_positive and not b.is_positive implies a * b = Int.from_nat(abs(a) * abs(b))
}

theorem mul_nonneg_nonneg_nonneg(a: Int, b: Int) {
    not a.is_negative and not b.is_negative implies not (a * b).is_negative
}

theorem mul_nonneg_nonpos_nonpos(a: Int, b: Int) {
    not a.is_negative and not b.is_positive implies not (a * b).is_positive
}

theorem mul_nonpos_nonneg_nonpos(a: Int, b: Int) {
    not a.is_positive and not b.is_negative implies not (a * b).is_positive
}

theorem mul_nonpos_nonpos_nonneg(a: Int, b: Int) {
    not a.is_positive and not b.is_positive implies not (a * b).is_negative
}

theorem mul_zero_left(a: Int) {
    0 * a = 0
}

theorem mul_comm(a: Int, b: Int) { a * b = b * a }

theorem mul_one_right(a: Int) { a * 1 = a }

theorem mul_one_left(a: Int) { 1 * a = a }

theorem mul_neg_left(a: Int, b: Int) { -a * b = -(a * b) }

theorem mul_neg_right(a: Int, b: Int) { a * -b = -(a * b) }

theorem mul_distrib_nonneg_right(a: Int, b: Int, c: Int) {
    not c.is_negative implies (a + b) * c = a * c + b * c
}

theorem mul_distrib_right(a: Int, b: Int, c: Int) { (a + b) * c = a * c + b * c }

theorem mul_distrib_left(a: Int, b: Int, c: Int) { a * (b + c) = a * b + a * c }

theorem mul_sub_distrib_right(a: Int, b: Int, c: Int) { (a - b) * c = a * c - b * c }

theorem mul_sub_distrib_left(a: Int, b: Int, c: Int) { a * (b - c) = a * b - a * c }

theorem abs_mul(a: Int, b: Int) { abs(a * b) = abs(a) * abs(b) }

// lattice.ac


theorem mul_from_nat(j: Nat, k: Nat) { Int.from_nat(j) * Int.from_nat(k) = Int.from_nat(j * k) }

theorem abs_zero_imp_zero(a: Int) {
    abs(a) = Nat.0 implies a = 0
}

theorem mul_zero_imp_factor_zero(a: Int, b: Int) { a * b = 0 implies a = 0 or b = 0 }

theorem mul_pos_pos(a: Int, b: Int) {
    a.is_positive and b.is_positive implies (a * b).is_positive
}

theorem mul_pos_neg(a: Int, b: Int) {
    a.is_positive and b.is_negative implies (a * b).is_negative
}

theorem mul_neg_pos(a: Int, b: Int) {
    a.is_negative and b.is_positive implies (a * b).is_negative
}

theorem mul_neg_neg(a: Int, b: Int) {
    a.is_negative and b.is_negative implies (a * b).is_positive
}

theorem mul_int_nat_nat_assoc(a: Int, j: Nat, k: Nat) { a.mul_nat(j * k) = (a.mul_nat(j)).mul_nat(k) }

theorem mul_int_int_nat_assoc(a: Int, b: Int, n: Nat) { (a * b).mul_nat(n) = a * (b).mul_nat(n) }

theorem mul_assoc(a: Int, b: Int, c: Int) { a * b * c = a * (b * c) }

// Comparison operators

/// `a <= b` when `(a - b)` is positive or zero.
instance Int: LTE {
    define lte(self, b: Int) -> Bool {
        (b - self).is_positive or self = b
    }
}

attributes Int {
    /// True if this integer divides b (equivalently, there exists d such that d * this = b).
    define divides(self, b: Int) -> Bool {
        exists(d: Int) { d * self = b }
    }
}


theorem int_is_reflexive {
    is_reflexive(Int.lte)
}

theorem lte_trans(a: Int, b: Int, c: Int) {
    a <= b and b <= c implies a <= c
}

theorem int_is_transitive {
    is_transitive(Int.lte)
}

theorem int_is_antisymmetric {
    is_antisymmetric(Int.lte)
}


instance Int: PartialOrder

theorem lt_not_ref(a: Int) { not (a < a) }

theorem lte_ref(a: Int) { a <= a }

theorem zero_lt_pos(a: Int) { a.is_positive implies 0 < a }

theorem sub_nonnegative_of_lte(a: Int, b: Int) {
    a <= b implies Int.0 <= b - a
}

theorem from_nat_pos(n: Nat) {
    Nat.0 < n implies Int.from_nat(n).is_positive
}

theorem neg_lt_zero(a: Int) {
    a.is_negative implies a < 0
}

theorem nonpos_lt_pos(a: Int, b: Int) {
    not a.is_positive and b.is_positive implies a < b
}

theorem neg_lt_nonneg(a: Int, b: Int) { a.is_negative and not b.is_negative implies a < b }

theorem nonpos_lte_nonneg(a: Int, b: Int) {
    not a.is_positive and not b.is_negative implies a <= b
}

theorem lte_abs(a: Int) { a <= Int.from_nat(abs(a)) }

theorem lt_add_left(a: Int, b: Int, c: Int) { b < c implies a + b < a + c }

theorem add_sub_lt_right_of_lt(a: Int, b: Int, c: Int) {
    a < b implies a + c - b < c
}

theorem lte_add_left(a: Int, b: Int, c: Int) { b <= c implies a + b <= a + c }

theorem lte_nonnegative_ints_implies_nats(n: Nat, m: Nat) {
    Int.from_nat(n) <= Int.from_nat(m) implies n <= m
}

theorem abs_add_nonneg(a: Int, b: Int) {
    not a.is_negative and not b.is_negative implies abs(a + b) = abs(a) + abs(b)
}

theorem abs_add_nonpos(a: Int, b: Int) {
    not a.is_positive and not b.is_positive implies abs(a + b) = abs(a) + abs(b)
}

// One case of the triangle inequality
theorem triangle_nonpos_lte_nonneg(a: Int, b: Int) {
    not a.is_negative and not b.is_positive and abs(b) <= abs(a) implies abs(a + b) <= abs(a) + abs(b)
}

theorem triangle_nonneg_nonpos(a: Int, b: Int) {
    not a.is_negative and not b.is_positive implies abs(a + b) <= abs(a) + abs(b)
}

theorem triangle_ineq(a: Int, b: Int) { abs(a + b) <= abs(a) + abs(b) }

theorem lt_mul_pos(a: Int, b: Int, c: Int) { a < b and c.is_positive implies a * c < b * c }

theorem lt_mul_neg(a: Int, b: Int, c: Int) { a < b and c.is_negative implies b * c < a * c }

theorem lt_trans(a: Int, b: Int, c: Int) { a < b and b < c implies a < c }

theorem lt_and_lte(a: Int, b: Int, c: Int) { a < b and b <= c implies a < c }

theorem lte_and_lt(a: Int, b: Int, c: Int) { a <= b and b < c implies a < c }

theorem lt_from_nat(j: Nat, k: Nat) {
    j < k implies Int.from_nat(j) < Int.from_nat(k)
}

theorem lte_from_nat(j: Nat, k: Nat) { j <= k implies Int.from_nat(j) <= Int.from_nat(k) }

// Units and dividing

define is_unit(a: Int) -> Bool { abs(a) = Nat.1 }

theorem two_units(u: Int) { is_unit(u) implies u = 1 or u = -(1) }

theorem unit_squared(u: Int) { is_unit(u) implies u * u = 1 }

theorem mul_units(u: Int, v: Int) { is_unit(u) and is_unit(v) implies is_unit(u * v) }

// Like the sign function, but we force it to be a unit, by considering 0 to have sign 1
define unit_sign(a: Int) -> Int {
    if a.is_negative {
        -(1)
    } else {
        1
    }
}

theorem unit_sign_is_unit(a: Int) { is_unit(unit_sign(a)) }

theorem abs_decomp(a: Int) { unit_sign(a) * Int.from_nat(abs(a)) = a }

theorem abs_alt_decomp(a: Int) { unit_sign(a) * a = Int.from_nat(abs(a)) }

theorem div_trans(a: Int, b: Int, c: Int) {
    a.divides(b) and b.divides(c) implies a.divides(c)
}

theorem div_imp_div_abs(a: Int, b: Int) {
    a.divides(b) implies abs(a).divides(abs(b))
}

theorem div_from_nat(j: Nat, k: Nat) {
    j.divides(k) implies Int.from_nat(j).divides(Int.from_nat(k))
}

theorem div_abs(a: Int) {
    a.divides(Int.from_nat(abs(a)))
}

theorem div_abs_imp_div(a: Int, b: Int) { abs(a).divides(abs(b)) implies a.divides(b) }

theorem abs_eq_imp_unit(a: Int, b: Int) {
    abs(a) = abs(b) implies exists(u: Int) { is_unit(u) and u * a = b }
}

theorem abs_eq_imp_div(a: Int, b: Int) { abs(a) = abs(b) implies a.divides(b) }

theorem div_pos_imp_lte(a: Int, b: Int) { a.divides(b) and b.is_positive implies a <= b }

// Theorems that relate to the GCD

attributes Int {
    /// The greatest common divisor of this integer and b.
    define gcd(self, b: Int) -> Int {
        Int.from_nat(abs(self).gcd(abs(b)))
    }
}

theorem gcd_nonneg(a: Int, b: Int) {
    not a.gcd(b).is_negative
}

theorem gcd_comm(a: Int, b: Int) { a.gcd(b) = b.gcd(a) }

theorem gcd_div_left(a: Int, b: Int) { a.gcd(b).divides(a) }

theorem gcd_div_right(a: Int, b: Int) { a.gcd(b).divides(b) }

theorem divides_gcd(a: Int, b: Int, d: Int) { d.divides(a) and d.divides(b) implies d.divides(a.gcd(b)) }

theorem gcd_pos(a: Int, b: Int) {
    a != 0 and b != 0 implies a.gcd(b).is_positive
}

theorem gcd_is_gcd(a: Int, b: Int, d: Int) {
    a != 0 and b != 0 and d.divides(a) and d.divides(b) implies d <= a.gcd(b)
}

// Theorems that relate to the LCM

attributes Int {
    /// The least common multiple of this integer and b, defined as the
    /// nonnegative integer with absolute value equal to the natural-number lcm
    /// of the two absolute values.
    define lcm(self, b: Int) -> Int {
        Int.from_nat(abs(self).lcm(abs(b)))
    }
}

/// The integer lcm is always nonnegative.
theorem int_lcm_nonneg(a: Int, b: Int) {
    not a.lcm(b).is_negative
}

/// Bridge: the integer lcm is the embedding of the natural lcm of the absolute values.
theorem int_lcm_from_nat_abs(a: Int, b: Int) {
    a.lcm(b) = Int.from_nat(abs(a).lcm(abs(b)))
}

/// Zero is absorbing for integer lcm on the left.
theorem int_lcm_zero_left(a: Int) {
    Int.0.lcm(a) = Int.0
}

/// Zero is absorbing for integer lcm on the right.
theorem int_lcm_zero_right(a: Int) {
    a.lcm(Int.0) = Int.0
}

/// Integer lcm is symmetric.
theorem int_lcm_comm(a: Int, b: Int) {
    a.lcm(b) = b.lcm(a)
}

/// Integer lcm is unchanged by negating the left input.
theorem int_lcm_neg_left(a: Int, b: Int) {
    (-a).lcm(b) = a.lcm(b)
}

/// Integer lcm is unchanged by negating the right input.
theorem int_lcm_neg_right(a: Int, b: Int) {
    a.lcm(-b) = a.lcm(b)
}

/// Integer lcm is unchanged by negating both inputs.
theorem int_lcm_neg_neg(a: Int, b: Int) {
    (-a).lcm(-b) = a.lcm(b)
}

/// The lcm of an integer with itself is its nonnegative associate.
theorem int_lcm_self(a: Int) {
    a.lcm(a) = Int.from_nat(abs(a))
}

/// The natural absolute value of the integer lcm is the natural lcm of absolute values.
theorem int_abs_lcm(a: Int, b: Int) {
    abs(a.lcm(b)) = abs(a).lcm(abs(b))
}

/// Defining product identity on the integers: gcd times lcm equals the
/// natural-number product of the absolute values, lifted to Int.
theorem int_gcd_mul_lcm(a: Int, b: Int) {
    a.gcd(b) * a.lcm(b) = Int.from_nat(abs(a) * abs(b))
}

/// Defining product identity in absolute-value form: gcd times lcm has the
/// same magnitude as the product a * b.
theorem int_gcd_mul_lcm_abs(a: Int, b: Int) {
    abs(a.gcd(b) * a.lcm(b)) = abs(a * b)
}

/// The left input divides the integer lcm.
theorem int_lcm_divides_left(a: Int, b: Int) {
    a.divides(a.lcm(b))
}

/// The right input divides the integer lcm.
theorem int_lcm_divides_right(a: Int, b: Int) {
    b.divides(a.lcm(b))
}

/// Universal property: every common integer multiple is divisible by the lcm.
theorem int_lcm_divides_of_common(a: Int, b: Int, c: Int) {
    a.divides(c) and b.divides(c) implies a.lcm(b).divides(c)
}

/// Integer lcm divisibility characterizes common integer multiples.
theorem int_lcm_divides_iff(a: Int, b: Int, c: Int) {
    a.lcm(b).divides(c) = (a.divides(c) and b.divides(c))
}

// Theorems about the span of a linear combination

define spans(a: Int, b: Int, c: Int) -> Bool {
    exists(d: Int, e: Int) {
        d * a + e * b = c
    }
}

theorem spans_zero(a: Int, b: Int) { spans(a, b, 0) }

theorem spans_left(a: Int, b: Int) { spans(a, b, a) }

theorem spans_comm(a: Int, b: Int, c: Int) { spans(a, b, c) implies spans(b, a, c) }

theorem spans_right(a: Int, b: Int) { spans(a, b, b) }

theorem spans_mul_left(a: Int, b: Int, c: Int) { spans(a, b, c * a) }

theorem spans_div_left(a: Int, b: Int, c: Int) { a.divides(c) implies spans(a, b, c) }

theorem spans_mul_right(a: Int, b: Int, c: Int) { spans(a, b, c * b) }

theorem spans_div_right(a: Int, b: Int, c: Int) { b.divides(c) implies spans(a, b, c) }

theorem spans_mul(a: Int, b: Int, c: Int, d: Int) { spans(a, b, c) implies spans(a, b, d * c) }

theorem spans_add(a: Int, b: Int, c: Int, d: Int) { spans(a, b, c) and spans(a, b, d) implies spans(a, b, c + d) }

theorem spans_negate(a: Int, b: Int, c: Int) { spans(a, b, c) implies spans(a, b, -(c)) }

let mod = Nat.mod

theorem spans_nat_mod(a: Int, b: Int, k: Nat, m: Nat) {
    spans(a, b, Int.from_nat(k)) and spans(a, b, Int.from_nat(m)) implies spans(a, b, Int.from_nat(mod(k, m)))
}

// Bezout's identity
theorem spans_gcd(a: Int, b: Int) { spans(a, b, a.gcd(b)) }

// More theorems analogous to nat theorems

theorem gcd_one_right(n: Int) {
    n.gcd(1) = 1
}

theorem gcd_one_left(n: Int) {
    1.gcd(n) = 1
}

theorem gcd_mult_left(a: Int, b: Int, m: Int) {
    Int.from_nat(abs(m)) * a.gcd(b) = (m * a).gcd(m * b)
}

theorem gcd_mult_right(a: Int, b: Int, m: Int) {
    a.gcd(b) * Int.from_nat(abs(m)) = (a * m).gcd(b * m)
}

theorem cofactor(a: Int, b: Int, af: Int, bf: Int) {
    (
        a.gcd(b) != 0 and
        af * a.gcd(b) = a and
        bf * a.gcd(b) = b
    ) implies af.gcd(bf) = 1
}

theorem gcd_nonzero_left(a: Int, b: Int) {
    a != 0 implies a.gcd(b) != 0
}

theorem gcd_nonzero_right(a: Int, b: Int) {
    b != 0 implies a.gcd(b) != 0
}

theorem lt_mul_both(a: Int, b: Int, c: Int) {
    a.is_positive and b < c implies a * b < a * c
}

theorem lte_mul_both(a: Int, b: Int, c: Int) {
    a.is_positive and b <= c implies a * b <= a * c
}

theorem lt_mul_both_neg(a: Int, b: Int, c: Int) {
    a.is_negative and b < c implies a * b > a * c
}

theorem lte_mul_both_neg(a: Int, b: Int, c: Int) {
    a.is_negative and b <= c implies a * b >= a * c
}

theorem lt_cancel_mul(a: Int, b: Int, c: Int) {
    a.is_positive and a * b < a * c implies b < c
}

theorem lte_cancel_mul(a: Int, b: Int, c: Int) {
    a.is_positive and a * b <= a * c implies b <= c
}

theorem lt_cancel_mul_neg(a: Int, b: Int, c: Int) {
    a.is_negative and a * b < a * c implies b > c
}

theorem lte_cancel_mul_neg(a: Int, b: Int, c: Int) {
    a.is_negative and a * b <= a * c implies b >= c
}

theorem mul_pos_cancel_left(a: Int, b: Int, c: Int) {
    a.is_positive and a * b = a * c implies b = c
}

theorem mul_pos_cancel_right(a: Int, b: Int, c: Int) {
    a.is_positive and b * a = c * a implies b = c
}

theorem mul_neg_cancel_left(a: Int, b: Int, c: Int) {
    a.is_negative and a * b = a * c implies b = c
}

theorem mul_neg_cancel_right(a: Int, b: Int, c: Int) {
    a.is_negative and b * a = c * a implies b = c
}

theorem mul_cancel_left(a: Int, b: Int, c: Int) {
    a != 0 and a * b = a * c implies b = c
}

theorem mul_cancel_right(a: Int, b: Int, c: Int) {
    a != 0 and b * a = c * a implies b = c
}

theorem divides_cancel_left(a: Int, b: Int, c: Int) {
    a != 0 and (a * b).divides(a * c) implies b.divides(c)
}

theorem divides_cancel_right(a: Int, b: Int, c: Int) {
    a != 0 and (b * a).divides(c * a) implies b.divides(c)
}

theorem divides_mul_left(a: Int, b: Int, m: Int) {
    a.divides(b) implies (m * a).divides(m * b)
}

theorem divides_mul_right(a: Int, b: Int, m: Int) {
    a.divides(b) implies (a * m).divides(b * m)
}

define is_prime(a: Int) -> Bool {
    abs(a).is_prime
}

theorem gcd_of_prime(p: Int, n: Int) {
    is_prime(p) implies p.gcd(n) = 1 or p.divides(n)
}

// Generalized version
theorem euclids_lemma(a: Int, b: Int, c: Int) {
    a.gcd(b) = 1 and a.divides(b * c) implies a.divides(c)
}

theorem euclids_lemma_prime(a: Int, b: Int, c: Int) {
    is_prime(a) and a.divides(b * c) implies a.divides(b) or a.divides(c)
}

theorem one_plus_one {
    1 + 1 = 2
}

theorem times_two(a: Int) {
    2 * a = a + a
}

// This is just the Nat division theorem ported up
theorem positive_division_theorem(m: Int, n: Int) {
    0 <= m and n.is_positive implies exists(q: Int, r: Int) {
        0 <= r and r < n and m = q * n + r
    }
}

// Dividing negative numbers is awkward, sometimes you're off by one
theorem negative_division_theorem(m: Int, n: Int) {
    m.is_negative and n.is_positive implies exists(q: Int, r: Int) {
        0 <= r and r < n and m = q * n + r
    }
}

// The entire division theorem is combining the two cases
theorem division_theorem(m: Int, n: Int) {
    n.is_positive implies exists(q: Int, r: Int) {
        0 <= r and r < n and m = q * n + r
    }
}

theorem add_lte(a: Int, b: Int, c: Int, d: Int) {
    a <= b and c <= d implies a + c <= b + d
}

theorem plus_abs_gte_zero(n: Int) {
    n + Int.from_nat(abs(n)) >= 0
}


instance Int: Semigroup


instance Int: Monoid

theorem exp_one(a: Int) {
    a.pow(Nat.1) = a
}

theorem exp_zero(a: Int) {
    a.pow(Nat.0) = 1
}

theorem exp_add(a: Int, b: Nat, c: Nat) {
    a.pow(b + c) = a.pow(b) * a.pow(c)
}

theorem exp_mul(a: Int, b: Nat, c: Nat) {
    a.pow(b * c) = a.pow(b).pow(c)
}

theorem zero_exp(n: Nat) {
    n != Nat.0 implies 0.pow(n) = 0
}

theorem one_exp(n: Nat) {
    1.pow(n) = 1
}

theorem pos_exp(a: Int, n: Nat) {
    a.is_positive implies a.pow(n).is_positive
}

theorem sq_eq_mul(a: Int) {
    a * a = a.pow(Nat.2)
}

theorem sq_pos(a: Int) {
    a != 0 implies a.pow(Nat.2).is_positive
}

theorem exp_abs_eq_abs_exp(a: Int, n: Nat) {
    abs(a.pow(n)) = abs(a).pow(n)
}

attributes Int {
    /// The absolute value of an integer.
    define abs(self) -> Int {
        Int.from_nat(abs(self))
    }
}

theorem member_abs_neg(a: Int) {
    a.is_negative implies a.abs = -a
}

theorem member_abs_nonneg(a: Int) {
    not a.is_negative implies a.abs = a
}

theorem member_abs_pos(a: Int) {
    a.is_positive implies a.abs = a
}

theorem member_abs_nonpos(a: Int) {
    not a.is_positive implies a.abs = -a
}

theorem member_triangle(a: Int, b: Int) {
    (a + b).abs <= a.abs + b.abs
}

// Connecting Int to its multiplicative algebraic structure.


instance Int: Semiring


instance Int: Ring


instance Int: CommSemigroup


instance Int: CommMonoid


instance Int: CommRing

theorem int_total(a: Int, b: Int) { a <= b or b <= a }

instance Int: LinearOrder


instance Int: AddLeftOrderedGroup
instance Int: AddOrderedGroup


instance Int: Meet {
    define meet(self, other: Int) -> Int {
        self.min(other)
    }
}

instance Int: Join {
    define join(self, other: Int) -> Int {
        self.max(other)
    }
}

theorem int_meet_lte_left(a: Int, b: Int) {
    a.meet(b) <= a
}

theorem int_meet_lte_right(a: Int, b: Int) {
    a.meet(b) <= b
}

theorem int_lte_meet_of_bounds(c: Int, a: Int, b: Int) {
    c <= a and c <= b implies c <= a.meet(b)
}

instance Int: MeetSemilattice

theorem int_lte_join_left(a: Int, b: Int) {
    a <= a.join(b)
}

theorem int_lte_join_right(a: Int, b: Int) {
    b <= a.join(b)
}

theorem int_join_lte_of_bounds(a: Int, b: Int, c: Int) {
    a <= c and b <= c implies a.join(b) <= c
}

instance Int: JoinSemilattice

instance Int: Lattice

theorem int_meet_join_distrib_left(a: Int, b: Int, c: Int) {
    a.meet(b.join(c)) = a.meet(b).join(a.meet(c))
}

theorem int_join_meet_distrib_left(a: Int, b: Int, c: Int) {
    a.join(b.meet(c)) = a.join(b).meet(a.join(c))
}

instance Int: DistribLattice

// int_arith.ac


/// Ten minus one is nine.
theorem ten_sub_one_eq_nine {
    Int.10 - Int.1 = Int.9
}

/// Multiplication of a negative embedded natural on the left.
theorem mul_neg_from_nat_left(m: Nat, n: Nat) {
    -Int.from_nat(m) * Int.from_nat(n) = -Int.from_nat(m * n)
}

/// Multiplication of a negative embedded natural on the right.
theorem mul_neg_from_nat_right(m: Nat, n: Nat) {
    Int.from_nat(m) * -Int.from_nat(n) = -Int.from_nat(m * n)
}

/// Multiplication of two negative embedded naturals.
theorem mul_neg_from_nat_neg(m: Nat, n: Nat) {
    -Int.from_nat(m) * -Int.from_nat(n) = Int.from_nat(m * n)
}

/// Three times three is nine.
theorem three_mul_three_eq_nine {
    Int.3 * Int.3 = Int.9
}
