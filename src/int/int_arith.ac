from int.int_base import Int, mul_neg_left, mul_neg_right
from int.lattice import mul_from_nat, mul_neg_neg
from nat import Nat, nine_plus_one

numerals Nat

/// Ten minus one is nine.
theorem ten_sub_one_eq_nine {
    Int.10 - Int.1 = Int.9
} by {
    Int.10 = Int.9 + Int.1
}

/// Multiplication of a negative embedded natural on the left.
theorem mul_neg_from_nat_left(m: Nat, n: Nat) {
    -Int.from_nat(m) * Int.from_nat(n) = -Int.from_nat(m * n)
} by {
    mul_from_nat(m, n)
    mul_neg_left(Int.from_nat(m), Int.from_nat(n))
}

/// Multiplication of a negative embedded natural on the right.
theorem mul_neg_from_nat_right(m: Nat, n: Nat) {
    Int.from_nat(m) * -Int.from_nat(n) = -Int.from_nat(m * n)
} by {
    mul_from_nat(m, n)
    mul_neg_right(Int.from_nat(m), Int.from_nat(n))
}

/// Multiplication of two negative embedded naturals.
theorem mul_neg_from_nat_neg(m: Nat, n: Nat) {
    -Int.from_nat(m) * -Int.from_nat(n) = Int.from_nat(m * n)
} by {
    mul_from_nat(m, n)
    mul_neg_neg(Int.from_nat(m), Int.from_nat(n))
}

/// Three times three is nine.
theorem three_mul_three_eq_nine {
    Int.3 * Int.3 = Int.9
} by {
    Int.3 * Int.3 = Int.from_nat(Nat.3 * Nat.3)
    Int.from_nat(Nat.3 * Nat.3) = Int.from_nat(Nat.9)
}
