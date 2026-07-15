from nat import divides_self
from int.lattice import Int, div_trans, div_imp_div_abs, div_abs_imp_div, div_abs,
    gcd_div_left, gcd_div_right, divides_gcd
from int.int_base import abs, abs_neg, sub_add_left
from int.int_divisibility import int_divides_self, int_divides_zero, int_neg_divides,
    int_divides_neg, int_divides_sub, int_divides_mul
numerals Int

/// Integer divisibility is transitive, restated with the `int_` prefix for
/// clients that import the integer divisibility API directly.
theorem int_divides_trans(a: Int, b: Int, c: Int) {
    a.divides(b) and b.divides(c) implies a.divides(c)
} by {
    if a.divides(b) and b.divides(c) {
        div_trans(a, b, c)
    }
}

/// Negating the divisor does not change integer divisibility.
theorem int_neg_divides_iff(a: Int, b: Int) {
    (-a).divides(b) = a.divides(b)
} by {
    if (-a).divides(b) {
        int_neg_divides(-a, b)
        a.divides(b)
    }
    if a.divides(b) {
        int_neg_divides(a, b)
        (-a).divides(b)
    }
}

/// Negating the dividend does not change integer divisibility.
theorem int_divides_neg_iff(a: Int, b: Int) {
    a.divides(-b) = a.divides(b)
} by {
    if a.divides(-b) {
        int_divides_neg(a, -b)
        a.divides(b)
    }
    if a.divides(b) {
        int_divides_neg(a, b)
        a.divides(-b)
    }
}

/// Divisibility is unchanged when both sides are negated.
theorem int_neg_divides_neg_iff(a: Int, b: Int) {
    (-a).divides(-b) = a.divides(b)
} by {
    int_neg_divides_iff(a, -b)
    int_divides_neg_iff(a, b)
}

/// Absolute values characterize divisibility over the integers.
theorem int_divides_iff_abs_divides(a: Int, b: Int) {
    a.divides(b) = abs(a).divides(abs(b))
} by {
    if a.divides(b) {
        div_imp_div_abs(a, b)
        abs(a).divides(abs(b))
    }
    if abs(a).divides(abs(b)) {
        div_abs_imp_div(a, b)
        a.divides(b)
    }
}

/// If `a | b`, then `a` also divides the nonnegative associate `|b|`.
theorem int_divides_abs_right(a: Int, b: Int) {
    a.divides(b) implies a.divides(Int.from_nat(abs(b)))
} by {
    if a.divides(b) {
        div_abs(b)
        div_trans(a, b, Int.from_nat(abs(b)))
        a.divides(Int.from_nat(abs(b)))
    }
}

/// An integer divides its nonnegative associate.
theorem int_self_divides_abs(a: Int) {
    a.divides(Int.from_nat(abs(a)))
} by {
    div_abs(a)
}

/// The nonnegative associate divides the original integer.
theorem int_abs_divides_self(a: Int) {
    Int.from_nat(abs(a)).divides(a)
} by {
    divides_self(abs(a))
    div_abs_imp_div(Int.from_nat(abs(a)), a)
}

/// Zero divides only zero.
theorem int_zero_divides_iff(a: Int) {
    Int.0.divides(a) = (a = Int.0)
} by {
    if Int.0.divides(a) {
        let c: Int satisfy { c * Int.0 = a }
        a = Int.0
    }
    if a = Int.0 {
        int_divides_zero(Int.0)
        Int.0.divides(a)
    }
}

/// Cancel a known divisible left summand from a divisible sum.
theorem int_divides_add_cancel_left(a: Int, b: Int, c: Int) {
    a.divides(b) and a.divides(b + c) implies a.divides(c)
} by {
    if a.divides(b) and a.divides(b + c) {
        int_divides_sub(a, b + c, b)
        sub_add_left(c, b)
        a.divides(c)
    }
}

/// Cancel a known divisible right summand from a divisible sum.
theorem int_divides_add_cancel_right(a: Int, b: Int, c: Int) {
    a.divides(c) and a.divides(b + c) implies a.divides(b)
} by {
    if a.divides(c) and a.divides(b + c) {
        int_divides_sub(a, b + c, c)
        sub_add_left(b, c)
        a.divides(b)
    }
}

/// Divisibility is preserved by multiplication on the left of the dividend.
theorem int_divides_mul_left_factor(a: Int, b: Int, c: Int) {
    a.divides(b) implies a.divides(c * b)
} by {
    if a.divides(b) {
        int_divides_mul(a, b, c)
        a.divides(c * b)
    }
}

/// Every integer divides any product where it is the left factor.
theorem int_divides_mul_self_left(a: Int, b: Int) {
    a.divides(a * b)
} by {
    int_divides_self(a)
    int_divides_mul(a, a, b)
}

/// Every integer divides any product where it is the right factor.
theorem int_divides_mul_self_right(a: Int, b: Int) {
    a.divides(b * a)
} by {
    int_divides_mul_self_left(a, b)
}

/// The gcd is unchanged by negating the left input.
theorem int_gcd_neg_left(a: Int, b: Int) {
    (-a).gcd(b) = a.gcd(b)
} by {
    abs_neg(a)
}

/// The gcd is unchanged by negating the right input.
theorem int_gcd_neg_right(a: Int, b: Int) {
    a.gcd(-b) = a.gcd(b)
} by {
    abs_neg(b)
}

/// The gcd is unchanged by negating both inputs.
theorem int_gcd_neg_neg(a: Int, b: Int) {
    (-a).gcd(-b) = a.gcd(b)
} by {
    int_gcd_neg_left(a, -b)
    int_gcd_neg_right(a, b)
}

/// The gcd divides the nonnegative associate of its left input.
theorem int_gcd_divides_abs_left(a: Int, b: Int) {
    a.gcd(b).divides(Int.from_nat(abs(a)))
} by {
    gcd_div_left(a, b)
    int_divides_abs_right(a.gcd(b), a)
}

/// The gcd divides the nonnegative associate of its right input.
theorem int_gcd_divides_abs_right(a: Int, b: Int) {
    a.gcd(b).divides(Int.from_nat(abs(b)))
} by {
    gcd_div_right(a, b)
    int_divides_abs_right(a.gcd(b), b)
}

/// Any common integer divisor divides the integer gcd.
theorem int_common_divisor_divides_gcd(a: Int, b: Int, d: Int) {
    d.divides(a) and d.divides(b) implies d.divides(a.gcd(b))
} by {
    if d.divides(a) and d.divides(b) {
        divides_gcd(a, b, d)
    }
}
