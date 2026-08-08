from int.lattice import Int
from int.lattice import is_prime
from int.int_base import abs, abs_from_nat
from nat import Nat, lte_antisymm
from algebra.ring.ring import mul_neg_neg, mul_neg_left
from int.lattice import euclids_lemma
numerals Int

/// Every integer divides itself.
theorem int_divides_self(a: Int) {
    a.divides(a)
} by {
}

/// One divides every integer.
theorem int_one_divides(a: Int) {
    Int.1.divides(a)
} by {
}

/// Every integer divides zero.
theorem int_divides_zero(a: Int) {
    a.divides(0)
} by {
}

/// If a divides b, so does the negation of a.
theorem int_neg_divides(a: Int, b: Int) {
    a.divides(b) implies (-a).divides(b)
} by {
    if a.divides(b) {
        let c: Int satisfy { c * a = b }
        mul_neg_neg(c, a)
        (-c) * (-a) = b
    }
}

/// If a divides b, then a also divides the negation of b.
theorem int_divides_neg(a: Int, b: Int) {
    a.divides(b) implies a.divides(-b)
} by {
    if a.divides(b) {
        let c: Int satisfy { c * a = b }
        mul_neg_left(c, a)
        (-c) * a = -b
    }
}

/// Divisibility on the integers is preserved by addition.
theorem int_divides_add(a: Int, b: Int, c: Int) {
    a.divides(b) and a.divides(c) implies a.divides(b + c)
} by {
    if a.divides(b) and a.divides(c) {
        let x: Int satisfy { x * a = b }
        let y: Int satisfy { y * a = c }
        Int.distrib_right(x, y, a)
        (x + y) * a = b + c
        exists(d: Int) { d * a = b + c }
    }
}

/// Divisibility on the integers is preserved by subtraction.
theorem int_divides_sub(a: Int, b: Int, c: Int) {
    a.divides(b) and a.divides(c) implies a.divides(b - c)
} by {
    if a.divides(b) and a.divides(c) {
        int_divides_neg(a, c)
        int_divides_add(a, b, -c)
        a.divides(b - c)
    }
}

/// Divisibility is preserved by multiplication on the right.
theorem int_divides_mul(a: Int, b: Int, c: Int) {
    a.divides(b) implies a.divides(b * c)
} by {
    if a.divides(b) {
        let x: Int satisfy { x * a = b }
        (x * c) * a = c * (x * a)
        (x * c) * a = b * c
    }
}

attributes Int {
    /// True if this integer and b have no common factor greater than one.
    define coprime(self, b: Int) -> Bool { self.gcd(b) = 1 }
}

/// Coprimality on the integers is symmetric in its arguments.
theorem int_coprime_comm(a: Int, b: Int) {
    a.coprime(b) implies b.coprime(a)
} by {
    if a.coprime(b) {
        b.gcd(a) = 1
    }
}

/// One is coprime with every integer on the left.
theorem int_coprime_one_left(a: Int) {
    Int.1.coprime(a)
} by {
}

/// One is coprime with every integer on the right.
theorem int_coprime_one_right(a: Int) {
    a.coprime(Int.1)
} by {
}

/// Euclid's lemma: if a is coprime to b and divides b * c, then a divides c.
theorem int_coprime_divides_of_divides_mul(a: Int, b: Int, c: Int) {
    a.coprime(b) and a.divides(b * c) implies a.divides(c)
} by {
    if a.coprime(b) and a.divides(b * c) {
        a.gcd(b) = 1
        euclids_lemma(a, b, c)
        a.divides(c)
    }
}

/// Bridge: a natural is coprime to b on Nat exactly when its embedding into
/// Int is coprime to the embedding of b on Int.
theorem nat_coprime_iff_int_coprime(a: Nat, b: Nat) {
    a.coprime(b) = Int.from_nat(a).coprime(Int.from_nat(b))
} by {
    Int.from_nat(a).gcd(Int.from_nat(b)) = Int.from_nat(a.gcd(b))
    Int.from_nat(a).coprime(Int.from_nat(b)) = (Int.from_nat(a.gcd(b)) = Int.from_nat(Nat.1))
}

/// Antisymmetry up to sign: if two integers divide each other and are nonzero, they have equal absolute value.
theorem int_divides_antisymm(a: Int, b: Int) {
    a != 0 and b != 0 and a.divides(b) and b.divides(a) implies abs(a) = abs(b)
} by {
    if a != 0 and b != 0 and a.divides(b) and b.divides(a) {
        a.divides(b) implies abs(a).divides(abs(b))
        b.divides(a) implies abs(b).divides(abs(a))
        abs(a).divides(abs(b))
        abs(b).divides(abs(a))
        abs(a) != Nat.0
        abs(b) != Nat.0
        abs(a).divides(abs(b)) implies abs(b) = Nat.0 or abs(a) <= abs(b)
        abs(b).divides(abs(a)) implies abs(a) = Nat.0 or abs(b) <= abs(a)
        abs(a) <= abs(b)
        abs(b) <= abs(a)
        abs(a) = abs(b)
    }
}

/// Bridge: a natural is prime exactly when its embedding into Int is prime.
theorem nat_is_prime_iff_int_is_prime(n: Nat) {
    n.is_prime = is_prime(Int.from_nat(n))
} by {
    abs_from_nat(n)
}

/// An integer is prime iff its absolute value is prime as a Nat. This is the
/// definitional unfold, restated as a top-level lemma so consumers can avoid
/// touching the underlying define.
theorem int_is_prime_iff_abs_is_prime(a: Int) {
    is_prime(a) = abs(a).is_prime
}
