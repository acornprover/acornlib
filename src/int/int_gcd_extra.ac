from int.lattice import Int, gcd_div_left, gcd_div_right, divides_gcd
from int.int_base import abs, abs_from_nat, abs_neg
from int.int_divisibility import int_divides_self, int_divides_antisymm
from nat import Nat
from nat import gcd_zero_left, gcd_zero_right, gcd_divides_left, gcd_divides_right
from nat import divides_self, divides_symm
numerals Int

/// The gcd of a natural with itself is itself.
theorem nat_gcd_self(a: Nat) {
    a.gcd(a) = a
} by {
    gcd_divides_left(a, a)
    divides_self(a)
    let d: Nat = a
    divides_symm(a.gcd(a), a)
}

/// Zero on the left makes gcd the absolute value of the other argument.
theorem int_gcd_zero_left(a: Int) {
    Int.0.gcd(a) = Int.from_nat(abs(a))
} by {
    gcd_zero_left(abs(a))
}

/// Zero on the right makes gcd the absolute value of the other argument.
theorem int_gcd_zero_right(a: Int) {
    a.gcd(Int.0) = Int.from_nat(abs(a))
} by {
    gcd_zero_right(abs(a))
}

/// The gcd of an integer with itself is the integer's absolute value.
theorem int_gcd_self(a: Int) {
    a.gcd(a) = Int.from_nat(abs(a))
} by {
    nat_gcd_self(abs(a))
}
