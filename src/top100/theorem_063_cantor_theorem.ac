from data.basic.functions import is_surjective_fn, surjective_fn_has_preimage
from data.basic.set import Set

/// AixBot-generated Freek Top 100 #63: Cantor's theorem.
theorem top100_063_cantor_theorem[T](f: T -> Set[T]) {
    not is_surjective_fn[T, Set[T]](f)
} by {
    let diagonal: Set[T] = Set[T].new(function(x: T) { not f(x).contains(x) })
    if is_surjective_fn[T, Set[T]](f) {
        surjective_fn_has_preimage[T, Set[T]](f, diagonal)
        let a: T satisfy { f(a) = diagonal }
        if diagonal.contains(a) {
            diagonal.contains(a) = not f(a).contains(a)
            f(a).contains(a) = diagonal.contains(a)
            not diagonal.contains(a)
            false
        }
        if not diagonal.contains(a) {
            diagonal.contains(a) = not f(a).contains(a)
            f(a).contains(a) = diagonal.contains(a)
            not f(a).contains(a)
            diagonal.contains(a)
            false
        }
        false
    }
}
