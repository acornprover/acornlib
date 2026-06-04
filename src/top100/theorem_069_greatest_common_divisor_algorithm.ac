from nat import Nat
from number_theory import gcd_pair, gcd_step_n, num_gcd_steps
from pair import Pair

/// Freek Top 100 #69: the Euclidean algorithm computes greatest common divisors.
theorem top100_069_greatest_common_divisor_algorithm(a: Nat, b: Nat) {
    exists(n: Nat) {
        gcd_step_n(Pair.new(a, b), n).second = Nat.0 and
        gcd_step_n(Pair.new(a, b), n).first = a.gcd(b) and
        a.gcd(b).divides(a) and
        a.gcd(b).divides(b) and
        forall(d: Nat) {
            d.divides(a) and d.divides(b) implies d.divides(a.gcd(b))
        }
    }
} by {
    let n = num_gcd_steps(Pair.new(a, b))
    gcd_step_n(Pair.new(a, b), n).second = Nat.0
    gcd_step_n(Pair.new(a, b), n).first = gcd_pair(Pair.new(a, b))
    gcd_pair(Pair.new(a, b)) = a.gcd(b)
    gcd_step_n(Pair.new(a, b), n).first = a.gcd(b)
    a.gcd(b).divides(a)
    a.gcd(b).divides(b)
    forall(d: Nat) {
        if d.divides(a) and d.divides(b) {
            d.divides(a.gcd(b))
        }
    }
}
