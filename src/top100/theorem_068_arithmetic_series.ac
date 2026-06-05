from list import partial
from nat import Nat
from real import Real
from real import nat_to_real

/// AixBot-generated Freek Top 100 #68: sum of an arithmetic series.
theorem nat_to_real_pred_suc(k: Nat) { nat_to_real(k.suc - Nat.1) = nat_to_real(k) } by {
    k.suc - Nat.1 = k
}

theorem half_linear(a: Real) {
    a * Real.one_half + Real.1 = (a + Real.1 + Real.1) * Real.one_half
} by {
    (a + Real.1 + Real.1) * Real.one_half =
        (a + Real.1) * Real.one_half + Real.1 * Real.one_half
    (a + Real.1) * Real.one_half = a * Real.one_half + Real.1 * Real.one_half
    Real.1 * Real.one_half = Real.one_half
    (a + Real.1 + Real.1) * Real.one_half =
        a * Real.one_half + Real.one_half + Real.one_half
    Real.one_half + Real.one_half = Real.1
    a * Real.one_half + Real.one_half + Real.one_half = a * Real.one_half + Real.1
}

theorem arithmetic_series_sum_suc(k: Nat) {
    partial[Real](nat_to_real, k.suc) = nat_to_real(k.suc) * nat_to_real(k) * Real.one_half
} by {
    define p(x: Nat) -> Bool {
        partial[Real](nat_to_real, x.suc) = nat_to_real(x.suc) * nat_to_real(x) * Real.one_half
    }

    partial[Real](nat_to_real, Nat.0.suc) = partial[Real](nat_to_real, Nat.0) + nat_to_real(Nat.0)
    partial[Real](nat_to_real, Nat.0) = Real.0
    nat_to_real(Nat.0) = Real.0
    partial[Real](nat_to_real, Nat.0.suc) = Real.0
    nat_to_real(Nat.0.suc) = Real.1
    nat_to_real(Nat.0.suc) * nat_to_real(Nat.0) = Real.0
    nat_to_real(Nat.0.suc) * nat_to_real(Nat.0) * Real.one_half = Real.0
    p(Nat.0)

    forall(x: Nat) {
        if p(x) {
            partial[Real](nat_to_real, x.suc) =
                nat_to_real(x.suc) * nat_to_real(x) * Real.one_half
            partial[Real](nat_to_real, x.suc.suc) =
                partial[Real](nat_to_real, x.suc) + nat_to_real(x.suc)
            partial[Real](nat_to_real, x.suc) + nat_to_real(x.suc) =
                nat_to_real(x.suc) * nat_to_real(x) * Real.one_half + nat_to_real(x.suc)
            partial[Real](nat_to_real, x.suc.suc) =
                nat_to_real(x.suc) * nat_to_real(x) * Real.one_half + nat_to_real(x.suc)
            nat_to_real(x.suc) * nat_to_real(x) * Real.one_half =
                nat_to_real(x.suc) * (nat_to_real(x) * Real.one_half)
            nat_to_real(x.suc) = nat_to_real(x.suc) * Real.1
            nat_to_real(x.suc) * (nat_to_real(x) * Real.one_half) +
                nat_to_real(x.suc) * Real.1 =
                nat_to_real(x.suc) * (nat_to_real(x) * Real.one_half + Real.1)
            partial[Real](nat_to_real, x.suc.suc) =
                nat_to_real(x.suc) * (nat_to_real(x) * Real.one_half + Real.1)
            nat_to_real(x) * Real.one_half + Real.1 =
                (nat_to_real(x) + Real.1 + Real.1) * Real.one_half
            nat_to_real(x.suc) = nat_to_real(x) + Real.1
            nat_to_real(x.suc.suc) = nat_to_real(x.suc) + Real.1
            nat_to_real(x.suc.suc) = nat_to_real(x) + Real.1 + Real.1
            (nat_to_real(x) + Real.1 + Real.1) * Real.one_half =
                nat_to_real(x.suc.suc) * Real.one_half
            nat_to_real(x) * Real.one_half + Real.1 =
                nat_to_real(x.suc.suc) * Real.one_half
            nat_to_real(x.suc) * (nat_to_real(x) * Real.one_half + Real.1) =
                nat_to_real(x.suc) * (nat_to_real(x.suc.suc) * Real.one_half)
            nat_to_real(x.suc) * (nat_to_real(x.suc.suc) * Real.one_half) =
                nat_to_real(x.suc) * nat_to_real(x.suc.suc) * Real.one_half
            partial[Real](nat_to_real, x.suc.suc) =
                nat_to_real(x.suc) * nat_to_real(x.suc.suc) * Real.one_half
            nat_to_real(x.suc) * nat_to_real(x.suc.suc) =
                nat_to_real(x.suc.suc) * nat_to_real(x.suc)
            nat_to_real(x.suc) * nat_to_real(x.suc.suc) * Real.one_half =
                nat_to_real(x.suc.suc) * nat_to_real(x.suc) * Real.one_half
            p(x.suc)
        }
    }
    p(k)
}

theorem top100_068_arithmetic_series(n: Nat) {
    partial[Real](nat_to_real, n) = nat_to_real(n) * nat_to_real(n - Nat.1) * Real.one_half
} by {
    define q(x: Nat) -> Bool {
        partial[Real](nat_to_real, x) = nat_to_real(x) * nat_to_real(x - Nat.1) * Real.one_half
    }
    partial[Real](nat_to_real, Nat.0) = Real.0
    nat_to_real(Nat.0) = Real.0
    nat_to_real(Nat.0) * nat_to_real(Nat.0 - Nat.1) = Real.0
    nat_to_real(Nat.0) * nat_to_real(Nat.0 - Nat.1) * Real.one_half = Real.0
    q(Nat.0)
    forall(x: Nat) {
        if q(x) {
            partial[Real](nat_to_real, x.suc) =
                nat_to_real(x.suc) * nat_to_real(x) * Real.one_half
            nat_to_real(x.suc - Nat.1) = nat_to_real(x)
            partial[Real](nat_to_real, x.suc) =
                nat_to_real(x.suc) * nat_to_real(x.suc - Nat.1) * Real.one_half
            q(x.suc)
        }
    }
    q(n)
}
