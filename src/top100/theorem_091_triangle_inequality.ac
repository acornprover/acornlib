from real import Real
from nat import Nat
from list import partial
from list import partial_one, partial_split_last
from order import lte_lt_trans
from algebra.add_ordered_group import add_inequality
from real import lt_mul_pos_right, lte_mul_nonneg_right, mul_nonneg
from real import finite_cauchy_schwarz, finite_dot, finite_square_sum,
    pointwise_product, pointwise_square, product_pair_square_rearrange,
    real_square_add_expanded
from geometry import Point2, point2_add_pair_rearrange, point2_norm_sq_add_expansion

numerals Real

/// The coordinate sequence of a real coordinate vector.
define point2_real_coord(p: Point2[Real], i: Nat) -> Real {
    if i = Nat.0 {
        p.x
    } else {
        p.y
    }
}

/// True when `r` is the Euclidean norm of a real coordinate vector.
define point2_real_norm(p: Point2[Real], r: Real) -> Bool {
    Real.0 <= r and r * r = p.norm_sq
}

/// Squaring reflects order between nonnegative real numbers.
theorem real_le_of_square_le_square_nonneg(a: Real, b: Real) {
    Real.0 <= a and Real.0 <= b and a * a <= b * b implies a <= b
} by {
    if Real.0 <= a and Real.0 <= b and a * a <= b * b {
        if b < a {
            Real.0 < a
            not b.is_negative
            b <= a
            lte_mul_nonneg_right(b, a, b)
            b * b <= a * b
            a.is_positive
            lt_mul_pos_right(b, a, a)
            b * a < a * a
            a * b = b * a
            a * b < a * a
            lte_lt_trans(b * b, a * b, a * a)
            b * b < a * a
            false
        }
    }
}

/// The finite dot product of the two coordinate sequences is the coordinate dot product.
theorem point2_real_finite_dot_two(a: Point2[Real], b: Point2[Real]) {
    finite_dot(point2_real_coord(a), point2_real_coord(b), Nat.2) = a.dot(b)
} by {
    let f = pointwise_product(point2_real_coord(a), point2_real_coord(b))
    finite_dot(point2_real_coord(a), point2_real_coord(b), Nat.2) = partial(f, Nat.2)
    partial_split_last(f, Nat.1)
    partial(f, Nat.2) = partial(f, Nat.1) + f(Nat.1)
    partial_one(f)
    partial(f, Nat.1) = f(Nat.0)
    f(Nat.0) = point2_real_coord(a, Nat.0) * point2_real_coord(b, Nat.0)
    f(Nat.1) = point2_real_coord(a, Nat.1) * point2_real_coord(b, Nat.1)
    point2_real_coord(a, Nat.0) = a.x
    point2_real_coord(b, Nat.0) = b.x
    point2_real_coord(a, Nat.1) = a.y
    point2_real_coord(b, Nat.1) = b.y
    a.dot(b) = a.x * b.x + a.y * b.y
}

/// The finite square sum of the two coordinate sequence is the squared coordinate norm.
theorem point2_real_finite_square_sum_two(a: Point2[Real]) {
    finite_square_sum(point2_real_coord(a), Nat.2) = a.norm_sq
} by {
    let f = pointwise_square(point2_real_coord(a))
    finite_square_sum(point2_real_coord(a), Nat.2) = partial(f, Nat.2)
    partial_split_last(f, Nat.1)
    partial(f, Nat.2) = partial(f, Nat.1) + f(Nat.1)
    partial_one(f)
    partial(f, Nat.1) = f(Nat.0)
    f(Nat.0) = point2_real_coord(a, Nat.0) * point2_real_coord(a, Nat.0)
    f(Nat.1) = point2_real_coord(a, Nat.1) * point2_real_coord(a, Nat.1)
    point2_real_coord(a, Nat.0) = a.x
    point2_real_coord(a, Nat.1) = a.y
    a.norm_sq = a.x * a.x + a.y * a.y
}

/// The dot product of real coordinate vectors is bounded by the product of their norms.
theorem point2_real_dot_le_norm_mul(a: Point2[Real], b: Point2[Real], an: Real, bn: Real) {
    point2_real_norm(a, an) and point2_real_norm(b, bn) implies a.dot(b) <= an * bn
} by {
    if point2_real_norm(a, an) and point2_real_norm(b, bn) {
        Real.0 <= an
        Real.0 <= bn
        an * an = a.norm_sq
        bn * bn = b.norm_sq
        finite_cauchy_schwarz(point2_real_coord(a), point2_real_coord(b), Nat.2)
        point2_real_finite_dot_two(a, b)
        point2_real_finite_square_sum_two(a)
        point2_real_finite_square_sum_two(b)
        a.dot(b) * a.dot(b) <= a.norm_sq * b.norm_sq
        a.norm_sq * b.norm_sq = (an * an) * (bn * bn)
        product_pair_square_rearrange(an, bn)
        (an * bn) * (an * bn) = (an * an) * (bn * bn)
        a.dot(b) * a.dot(b) <= (an * bn) * (an * bn)
        an >= Real.0
        bn >= Real.0
        mul_nonneg(an, bn)
        an * bn >= Real.0
        Real.0 <= an * bn
        if an * bn < a.dot(b) {
            an * bn >= Real.0
            Real.0 <= an * bn
            lte_lt_trans(Real.0, an * bn, a.dot(b))
            Real.0 < a.dot(b)
            Real.0 <= a.dot(b)
            real_le_of_square_le_square_nonneg(a.dot(b), an * bn)
            false
        }
        not an * bn < a.dot(b)
        a.dot(b) <= an * bn
    }
}

/// Freek Top 100 #91, specialized to the Euclidean norm on `R^2`.
theorem top100_091_triangle_inequality_r2(
    a: Point2[Real], b: Point2[Real], an: Real, bn: Real, abn: Real
) {
    point2_real_norm(a, an) and point2_real_norm(b, bn) and point2_real_norm(a.add(b), abn)
        implies abn <= an + bn
} by {
    if point2_real_norm(a, an) and point2_real_norm(b, bn) and point2_real_norm(a.add(b), abn) {
        Real.0 <= an
        Real.0 <= bn
        Real.0 <= abn
        Real.0 + Real.0 <= an + bn
        Real.0 + Real.0 = Real.0
        Real.0 <= an + bn
        an * an = a.norm_sq
        bn * bn = b.norm_sq
        abn * abn = a.add(b).norm_sq
        point2_norm_sq_add_expansion(a, b)
        a.add(b).norm_sq = a.norm_sq + b.norm_sq + a.dot(b) + a.dot(b)
        point2_real_dot_le_norm_mul(a, b, an, bn)
        a.dot(b) <= an * bn
        a.dot(b) + a.dot(b) <= an * bn + an * bn
        a.norm_sq + b.norm_sq = an * an + bn * bn
        a.norm_sq + b.norm_sq <= an * an + bn * bn
        a.norm_sq + b.norm_sq + a.dot(b) + a.dot(b) = (a.norm_sq + b.norm_sq) + (a.dot(b) + a.dot(b))
        an * an + bn * bn + an * bn + an * bn = (an * an + bn * bn) + (an * bn + an * bn)
        add_inequality(a.norm_sq + b.norm_sq, an * an + bn * bn, a.dot(b) + a.dot(b), an * bn + an * bn)
        (a.norm_sq + b.norm_sq) + (a.dot(b) + a.dot(b)) <= (an * an + bn * bn) + (an * bn + an * bn)
        a.norm_sq + b.norm_sq + a.dot(b) + a.dot(b) <= an * an + bn * bn + an * bn + an * bn
        real_square_add_expanded(an, bn)
        (an + bn) * (an + bn) = an * an + an * bn + (an * bn + bn * bn)
        point2_add_pair_rearrange(an * an, bn * bn, an * bn, an * bn)
        (an * an + bn * bn) + (an * bn + an * bn) = (an * an + an * bn) + (bn * bn + an * bn)
        bn * bn + an * bn = an * bn + bn * bn
        an * an + bn * bn + an * bn + an * bn = an * an + an * bn + (an * bn + bn * bn)
        an * an + bn * bn + an * bn + an * bn = (an + bn) * (an + bn)
        abn * abn <= (an + bn) * (an + bn)
        real_le_of_square_le_square_nonneg(abn, an + bn)
        abn <= an + bn
    }
}
