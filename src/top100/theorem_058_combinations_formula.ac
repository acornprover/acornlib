from combinatorics import binom, choose_add
from nat import Nat

/// Freek Top 100 #58: formula for the number of combinations.
/// AixBot-generated; manually reviewed and verified with Acorn.
theorem top100_058_combinations_formula(n: Nat, k: Nat) {
    k <= n implies n.binom(k) * k.factorial * (n - k).factorial = n.factorial
} by {
    choose_add(k, n - k)
    k + (n - k) = n
}
