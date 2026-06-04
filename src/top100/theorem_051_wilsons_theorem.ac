from nat import Nat
from number_theory import prime_imp_wilson_factorial_congr,
    wilson_factorial_congr_imp_prime

numerals Nat

/// Freek Top 100 #51: Wilson's theorem.
theorem top100_051_wilsons_theorem(p: Nat) {
    p != Nat.1 implies
        (p.is_prime = (p - Nat.1).factorial.congr_mod(p - Nat.1, p))
} by {
    if p != Nat.1 {
        if p.is_prime {
            prime_imp_wilson_factorial_congr(p)
            (p - Nat.1).factorial.congr_mod(p - Nat.1, p)
        }
        if (p - Nat.1).factorial.congr_mod(p - Nat.1, p) {
            wilson_factorial_congr_imp_prime(p)
            p.is_prime
        }
        p.is_prime = (p - Nat.1).factorial.congr_mod(p - Nat.1, p)
    }
}
