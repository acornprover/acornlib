from int import Int, spans

numerals Int

/// Bezout's identity: the integer gcd is an integer linear combination.
theorem top100_060_bezout_identity(a: Int, b: Int) {
    exists(x: Int, y: Int) {
        a.gcd(b) = a * x + b * y
    }
} by {
    spans(a, b, a.gcd(b))
    let (u: Int, v: Int) satisfy {
        u * a + v * b = a.gcd(b)
    }
    a * u = u * a
    b * v = v * b
    a * u + b * v = a.gcd(b)
    a.gcd(b) = a * u + b * v
}

/// Freek Top 100 #60: Bezout's theorem for integer gcds.
theorem top100_060_bezouts_theorem(a: Int, b: Int) {
    exists(d: Int) {
        not d.is_negative and
        d.divides(a) and
        d.divides(b) and
        exists(x: Int, y: Int) {
            d = a * x + b * y
        }
    }
} by {
    let d = a.gcd(b)
    not d.is_negative
    d.divides(a)
    d.divides(b)
    top100_060_bezout_identity(a, b)
}
