from int import Int, abs, gcd_of_prime, is_prime
from nat import Nat
from number_theory import no_proper_divisor_imp_prime
from rat import Rat, is_reduced

/// Two is nonzero as an integer.
theorem int_two_ne_zero { Int.1 + Int.1 != Int.0 }

/// Two does not divide one as an integer.
theorem int_two_not_divides_one { not (Int.1 + Int.1).divides(Int.1) } by {
    if (Int.1 + Int.1).divides(Int.1) {
        abs(Int.1 + Int.1).divides(abs(Int.1))
        abs(Int.1 + Int.1) = Nat.2
        abs(Int.1) = Nat.1
        Nat.2.divides(Nat.1)
        Nat.2 = Nat.1
        false
    }
}

/// The numerator and denominator of a rational number are reduced.
theorem rat_num_denom_reduced(r: Rat) { is_reduced(r.num, r.denom) }

/// Reduced integers have no common factor two.
theorem reduced_no_common_factor_two(a: Int, b: Int) {
    is_reduced(a, b) implies not ((Int.1 + Int.1).divides(a) and (Int.1 + Int.1).divides(b))
} by {
    if (Int.1 + Int.1).divides(a) and (Int.1 + Int.1).divides(b) {
        (Int.1 + Int.1).divides(a.gcd(b))
        a.gcd(b) = Int.1
        (Int.1 + Int.1).divides(Int.1)
        false
    }
}

/// Two is prime as a natural number.
theorem nat_two_prime { Nat.2.is_prime } by {
    Nat.1 < Nat.2
    forall(k: Nat) {
        if Nat.1 < k and k < Nat.2 {
            false
        }
    }
    no_proper_divisor_imp_prime(Nat.2)
}

/// Two is prime as an integer.
theorem int_two_prime { is_prime(Int.2) } by {
    nat_two_prime
    Nat.2.is_prime = is_prime(Int.from_nat(Nat.2))
    Int.from_nat(Nat.2) = Int.2
    is_prime(Int.2)
}

/// If two divides an integer square, then two divides the integer.
theorem int_two_divides_square(a: Int) {
    (Int.1 + Int.1).divides(a * a) implies (Int.1 + Int.1).divides(a)
} by {
    Int.1 + Int.1 = Int.2
    is_prime(Int.2)
    gcd_of_prime(Int.2, a)
    if Int.2.gcd(a) = Int.1 {
        Int.2.gcd(a) * Int.from_nat(abs(a)) = (Int.2 * a).gcd(a * a)
        Int.2.divides(Int.2 * a)
        Int.2.divides(a * a)
        Int.2.divides((Int.2 * a).gcd(a * a))
        Int.2.divides(Int.from_nat(abs(a)))
        Int.2.divides(a)
    }
}

/// If one square is twice another square, then two divides the left square root.
theorem int_two_divides_left_of_sq_eq_two_mul_sq(a: Int, b: Int) {
    a * a = (Int.1 + Int.1) * (b * b) implies (Int.1 + Int.1).divides(a)
} by {
    (b * b) * (Int.1 + Int.1) = a * a
    (Int.1 + Int.1).divides(a * a)
    int_two_divides_square(a)
}

/// The integer two, expressed as one plus one.
define int_two(dummy: Int) -> Int { Int.1 + Int.1 }

/// If one square is twice another square, then two divides the right square root.
theorem int_two_divides_right_of_sq_eq_two_mul_sq(a: Int, b: Int) {
    a * a = (Int.1 + Int.1) * (b * b) implies (Int.1 + Int.1).divides(b)
} by {
    int_two(Int.0) = Int.1 + Int.1
    (Int.1 + Int.1).divides(a)
    int_two(Int.0).divides(a)
    let c: Int satisfy { c * int_two(Int.0) = a }
    c * int_two(Int.0) = int_two(Int.0) * c
    int_two(Int.0) * c = a
    a = int_two(Int.0) * c
    a * a = (int_two(Int.0) * c) * (int_two(Int.0) * c)
    (int_two(Int.0) * c) * (int_two(Int.0) * c) = int_two(Int.0) * (c * (int_two(Int.0) * c))
    c * (int_two(Int.0) * c) = (c * int_two(Int.0)) * c
    c * int_two(Int.0) = int_two(Int.0) * c
    (c * int_two(Int.0)) * c = (int_two(Int.0) * c) * c
    (int_two(Int.0) * c) * c = int_two(Int.0) * (c * c)
    c * (int_two(Int.0) * c) = int_two(Int.0) * (c * c)
    int_two(Int.0) * (c * (int_two(Int.0) * c)) = int_two(Int.0) * (int_two(Int.0) * (c * c))
    (int_two(Int.0) * c) * (int_two(Int.0) * c) = int_two(Int.0) * (int_two(Int.0) * (c * c))
    a * a = int_two(Int.0) * (int_two(Int.0) * (c * c))
    (Int.1 + Int.1) * (b * b) = int_two(Int.0) * (b * b)
    a * a = int_two(Int.0) * (b * b)
    int_two(Int.0) * (int_two(Int.0) * (c * c)) = int_two(Int.0) * (b * b)
    int_two(Int.0) != Int.0
    int_two(Int.0) * (int_two(Int.0) * (c * c)) = int_two(Int.0) * (b * b) implies int_two(Int.0) * (c * c) = b * b
    int_two(Int.0) * (c * c) = b * b
    b * b = int_two(Int.0) * (c * c)
    int_two(Int.0).divides(b * b)
    (Int.1 + Int.1).divides(b * b)
    (Int.1 + Int.1).divides(b)
}

/// Multiplying a rational square by two denominator factors may be rearranged as a square.
theorem rat_square_denoms_rearrange(r: Rat) {
    (r * r) * Rat.from_int(r.denom) * Rat.from_int(r.denom) =
    (r * Rat.from_int(r.denom)) * (r * Rat.from_int(r.denom))
} by {
    let d = Rat.from_int(r.denom)
    (r * r) * d * d = r * r * d * d
    r * r * d = r * (r * d)
    r * (r * d) = r * (d * r)
    r * (d * r) = r * d * r
    r * r * d = r * d * r
    r * r * d * d = r * d * r * d
    r * d * r * d = (r * d) * (r * d)
    (r * r) * d * d = (r * d) * (r * d)
}

/// The integer two is one plus one.
theorem int_two_eq_one_plus_one { Int.2 = Int.1 + Int.1 }

/// A rational square equal to two gives the corresponding integer numerator-denominator equation.
theorem rat_square_eq_two_num_denom(r: Rat) {
    r * r = Rat.2 implies r.num * r.num = (Int.1 + Int.1) * (r.denom * r.denom)
} by {
    r * Rat.from_int(r.denom) = Rat.from_int(r.num)
    (r * r) * Rat.from_int(r.denom) * Rat.from_int(r.denom) =
    (r * Rat.from_int(r.denom)) * (r * Rat.from_int(r.denom))
    (r * Rat.from_int(r.denom)) * (r * Rat.from_int(r.denom)) =
    Rat.from_int(r.num) * Rat.from_int(r.num)
    Rat.from_int(r.num) * Rat.from_int(r.num) = Rat.from_int(r.num * r.num)
    (r * r) * Rat.from_int(r.denom) * Rat.from_int(r.denom) = Rat.from_int(r.num * r.num)

    Rat.2 = Rat.from_int(Int.2)
    Rat.2 * Rat.from_int(r.denom) = Rat.from_int(Int.2) * Rat.from_int(r.denom)
    Rat.from_int(Int.2) * Rat.from_int(r.denom) = Rat.from_int(Int.2 * r.denom)
    Rat.2 * Rat.from_int(r.denom) = Rat.from_int(Int.2 * r.denom)
    Rat.2 * Rat.from_int(r.denom) * Rat.from_int(r.denom) =
    Rat.from_int(Int.2 * r.denom) * Rat.from_int(r.denom)
    Rat.from_int(Int.2 * r.denom) * Rat.from_int(r.denom) = Rat.from_int((Int.2 * r.denom) * r.denom)
    (Int.2 * r.denom) * r.denom = Int.2 * (r.denom * r.denom)
    Rat.from_int((Int.2 * r.denom) * r.denom) = Rat.from_int(Int.2 * (r.denom * r.denom))
    Rat.2 * Rat.from_int(r.denom) * Rat.from_int(r.denom) =
    Rat.from_int(Int.2 * (r.denom * r.denom))
    Int.2 = Int.1 + Int.1
    Int.2 * (r.denom * r.denom) = (Int.1 + Int.1) * (r.denom * r.denom)
    Rat.2 * Rat.from_int(r.denom) * Rat.from_int(r.denom) =
    Rat.from_int((Int.1 + Int.1) * (r.denom * r.denom))

    (r * r) * Rat.from_int(r.denom) * Rat.from_int(r.denom) =
    Rat.2 * Rat.from_int(r.denom) * Rat.from_int(r.denom)
    Rat.from_int(r.num * r.num) = Rat.from_int((Int.1 + Int.1) * (r.denom * r.denom))
    r.num * r.num = (Int.1 + Int.1) * (r.denom * r.denom)
}

/// AixBot-generated Freek Top 100 #1: irrationality of the square root of 2.
theorem top100_001_sqrt2_irrational_core(r: Rat) { r * r != Rat.2 } by {
    if r * r = Rat.2 {
        r.num * r.num = (Int.1 + Int.1) * (r.denom * r.denom)
        (Int.1 + Int.1).divides(r.num)
        (Int.1 + Int.1).divides(r.denom)
        is_reduced(r.num, r.denom)
        not ((Int.1 + Int.1).divides(r.num) and (Int.1 + Int.1).divides(r.denom))
        false
    }
}
