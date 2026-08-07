from data.basic.functions import is_bijection_fn, is_injective_fn, is_surjective_fn
from top100.theorem_025_schroeder_bernstein import schroeder_bernstein_map,
    schroeder_bernstein_map_injective, schroeder_bernstein_map_surjective

/// The concrete Schroeder-Bernstein map is a bijection when both supplied maps
/// are injective and a default codomain element is fixed.
theorem schroeder_bernstein_map_bijection_with_default[A, B](
    f: A -> B, g: B -> A, default: B
) {
    is_injective_fn(f) and is_injective_fn(g) implies
        is_bijection_fn(schroeder_bernstein_map(f, g, default))
} by {
    if is_injective_fn(f) and is_injective_fn(g) {
        schroeder_bernstein_map_injective(f, g, default)
        is_injective_fn(schroeder_bernstein_map(f, g, default))
        schroeder_bernstein_map_surjective(f, g, default)
        is_surjective_fn(schroeder_bernstein_map(f, g, default))
        is_bijection_fn(schroeder_bernstein_map(f, g, default))
    }
}

/// Witness-shaped version: the canonical Schroeder-Bernstein map itself is the
/// bijection supplied by the construction with a default codomain element.
theorem schroeder_bernstein_bijection_structure_with_default[A, B](
    f: A -> B, g: B -> A, default: B
) {
    is_injective_fn(f) and is_injective_fn(g) implies exists(h: A -> B) {
        h = schroeder_bernstein_map(f, g, default) and is_bijection_fn(h)
    }
} by {
    if is_injective_fn(f) and is_injective_fn(g) {
        schroeder_bernstein_map_bijection_with_default(f, g, default)
        is_bijection_fn(schroeder_bernstein_map(f, g, default))
        exists(h: A -> B) {
            h = schroeder_bernstein_map(f, g, default) and is_bijection_fn(h)
        }
    }
}
