from data.basic.functions import function_eq_transport_injective_fn, injective_fn_eq, is_bijection_fn,
    is_injective_fn, is_surjective_fn
from nat import Nat
from data.basic.witness import choose_or_default, choose_or_default_spec,
    choose_or_default_unique_eq, exists_unique, exists_unique_intro

/// A map is injective when equal outputs force equal inputs.
theorem injective_fn_intro_from_eq[T, U](h: T -> U) {
    (forall(x: T, y: T) {
        h(x) = h(y) implies x = y
    }) implies is_injective_fn(h)
} by {
    if forall(x: T, y: T) { h(x) = h(y) implies x = y } {
        is_injective_fn(h)
    }
}

/// A witness-shaped inhabitedness predicate for generic types.
define has_element_witness[T](x: T) -> Bool {
    x = x
}

/// A function into an empty type is surjective.
theorem function_to_empty_is_surjective[T, U](f: T -> U) {
    not exists(y: U) { has_element_witness(y) } implies is_surjective_fn(f)
} by {
    if not exists(y: U) { has_element_witness(y) } {
        if not is_surjective_fn(f) {
            let y: U satisfy {
                not exists(x: T) {
                    f(x) = y
                }
            }
            exists(witness: U) {
                witness = y
            }
            exists(witness: U) {
                witness = y and has_element_witness(witness)
            }
            false
        }
        is_surjective_fn(f)
    }
}

/// True if `b` is a preimage of `a` under `g`.
define fiber_predicate[A, B](g: B -> A, a: A, b: B) -> Bool {
    g(b) = a
}

/// A chosen preimage under `g`, with the given default outside the image.
define injection_inverse[A, B](g: B -> A, default: B, a: A) -> B {
    choose_or_default(fiber_predicate(g, a), default)
}

/// The chosen preimage maps to the target whenever the target lies in the image.
theorem injection_inverse_maps_to_target[A, B](g: B -> A, default: B, a: A) {
    exists(b: B) {
        g(b) = a
    } implies g(injection_inverse(g, default, a)) = a
} by {
    if exists(b: B) { g(b) = a } {
        exists(b: B) {
            fiber_predicate(g, a, b)
        }
        choose_or_default_spec(fiber_predicate(g, a), default)
        fiber_predicate(g, a, injection_inverse(g, default, a))
        g(injection_inverse(g, default, a)) = a
    }
}

/// An injective map has at most one preimage of a point.
theorem fiber_predicate_unique_of_injective[A, B](g: B -> A, a: A, x: B, y: B) {
    is_injective_fn(g) and fiber_predicate(g, a, x) and fiber_predicate(g, a, y)
        implies x = y
} by {
    if is_injective_fn(g) and fiber_predicate(g, a, x) and fiber_predicate(g, a, y) {
        g(x) = a
        g(y) = a
        g(x) = g(y)
        injective_fn_eq(g, x, y)
        x = y
    }
}

/// The fiber over an image point of an injective map has exactly one element.
theorem fiber_image_exists_unique_of_injective[A, B](g: B -> A, b: B) {
    is_injective_fn(g) implies exists_unique(fiber_predicate(g, g(b)))
} by {
    if is_injective_fn(g) {
        fiber_predicate(g, g(b), b)
        forall(y: B) {
            if fiber_predicate(g, g(b), y) {
                fiber_predicate_unique_of_injective(g, g(b), y, b)
                y = b
            }
        }
        exists_unique_intro(fiber_predicate(g, g(b)), b)
        exists_unique(fiber_predicate(g, g(b)))
    }
}

/// On the image of an injective map, the chosen preimage is the original point.
theorem injection_inverse_apply_image[A, B](g: B -> A, default: B, b: B) {
    is_injective_fn(g) implies injection_inverse(g, default, g(b)) = b
} by {
    if is_injective_fn(g) {
        fiber_image_exists_unique_of_injective(g, b)
        exists_unique(fiber_predicate(g, g(b)))
        fiber_predicate(g, g(b), b)
        choose_or_default_unique_eq(fiber_predicate(g, g(b)), default, b)
        injection_inverse(g, default, g(b)) = b
    }
}

/// True if `a` lies in the image of `g`.
define image_contains[A, B](g: B -> A, a: A) -> Bool {
    exists(b: B) {
        g(b) = a
    }
}

/// Applying `g` to a point gives a point in the image of `g`.
theorem image_contains_apply[A, B](g: B -> A, b: B) {
    image_contains(g, g(b))
} by {
    exists(preimage: B) {
        g(preimage) = g(b)
    }
}

/// One step of the Schroeder-Bernstein chain construction.
define schroeder_bernstein_step[A, B](f: A -> B, g: B -> A, a: A) -> A {
    g(f(a))
}

/// The `n`-fold forward iterate of `g ∘ f` from `a`.
define schroeder_bernstein_iter[A, B](f: A -> B, g: B -> A, a: A, n: Nat) -> A {
    match n {
        Nat.zero {
            a
        }
        Nat.suc(pred) {
            schroeder_bernstein_step(f, g, schroeder_bernstein_iter(f, g, a, pred))
        }
    }
}

/// The successor iterate unfolds by one forward step.
theorem schroeder_bernstein_iter_suc[A, B](f: A -> B, g: B -> A, a: A, n: Nat) {
    schroeder_bernstein_iter(f, g, a, n.suc) =
        g(f(schroeder_bernstein_iter(f, g, a, n)))
}

/// True if `a` is reachable from outside the image of `g` by finitely many
/// applications of `g ∘ f`.
define schroeder_bernstein_chain[A, B](f: A -> B, g: B -> A, a: A) -> Bool {
    exists(start: A, n: Nat) {
        not image_contains(g, start) and
        schroeder_bernstein_iter(f, g, start, n) = a
    }
}

/// Every point outside the image of `g` begins a Schroeder-Bernstein chain.
theorem schroeder_bernstein_chain_base[A, B](f: A -> B, g: B -> A, a: A) {
    not image_contains(g, a) implies schroeder_bernstein_chain(f, g, a)
} by {
    if not image_contains(g, a) {
        schroeder_bernstein_iter(f, g, a, Nat.0) = a
        exists(start: A, n: Nat) {
            not image_contains(g, start) and
            schroeder_bernstein_iter(f, g, start, n) = a
        }
    }
}

/// The chain set is closed under the forward map `g ∘ f`.
theorem schroeder_bernstein_chain_forward[A, B](f: A -> B, g: B -> A, a: A) {
    schroeder_bernstein_chain(f, g, a) implies
        schroeder_bernstein_chain(f, g, g(f(a)))
} by {
    if schroeder_bernstein_chain(f, g, a) {
        let (start: A, n: Nat) satisfy {
            not image_contains(g, start) and
            schroeder_bernstein_iter(f, g, start, n) = a
        }
        schroeder_bernstein_iter_suc(f, g, start, n)
        schroeder_bernstein_iter(f, g, start, n.suc) =
            g(f(schroeder_bernstein_iter(f, g, start, n)))
        schroeder_bernstein_iter(f, g, start, n.suc) = g(f(a))
        exists(next_start: A, next_n: Nat) {
            not image_contains(g, next_start) and
            schroeder_bernstein_iter(f, g, next_start, next_n) = g(f(a))
        }
    }
}

/// If a point is not in the chain set, then it lies in the image of `g`.
theorem not_chain_imp_image_contains[A, B](f: A -> B, g: B -> A, a: A) {
    not schroeder_bernstein_chain(f, g, a) implies image_contains(g, a)
} by {
    if not schroeder_bernstein_chain(f, g, a) {
        if not image_contains(g, a) {
            schroeder_bernstein_chain_base(f, g, a)
            schroeder_bernstein_chain(f, g, a)
            false
        }
    }
}

/// A chained image point has a chained predecessor under `f`.
theorem schroeder_bernstein_chain_backward_image[A, B](f: A -> B, g: B -> A, y: B) {
    is_injective_fn(g) and schroeder_bernstein_chain(f, g, g(y)) implies
        exists(a: A) {
            schroeder_bernstein_chain(f, g, a) and f(a) = y
        }
} by {
    if is_injective_fn(g) and schroeder_bernstein_chain(f, g, g(y)) {
        let (start: A, n: Nat) satisfy {
            not image_contains(g, start) and
            schroeder_bernstein_iter(f, g, start, n) = g(y)
        }
        match n {
            Nat.zero {
                schroeder_bernstein_iter(f, g, start, Nat.0) = start
                start = g(y)
                image_contains_apply(g, y)
                image_contains(g, start)
                false
            }
            Nat.suc(pred) {
                schroeder_bernstein_iter_suc(f, g, start, pred)
                schroeder_bernstein_iter(f, g, start, pred.suc) =
                    g(f(schroeder_bernstein_iter(f, g, start, pred)))
                g(f(schroeder_bernstein_iter(f, g, start, pred))) = g(y)
                injective_fn_eq(g, f(schroeder_bernstein_iter(f, g, start, pred)), y)
                f(schroeder_bernstein_iter(f, g, start, pred)) = y
                schroeder_bernstein_chain(f, g, schroeder_bernstein_iter(f, g, start, pred))
                exists(a: A) {
                    schroeder_bernstein_chain(f, g, a) and f(a) = y
                }
            }
        }
    }
}

/// The Schroeder-Bernstein map: use `f` on the forward chains and use the
/// partial inverse of `g` elsewhere.
define schroeder_bernstein_map[A, B](f: A -> B, g: B -> A, default: B, a: A) -> B {
    if schroeder_bernstein_chain(f, g, a) {
        f(a)
    } else {
        injection_inverse(g, default, a)
    }
}

/// On the chain set, the Schroeder-Bernstein map agrees with `f`.
theorem schroeder_bernstein_map_of_chain[A, B](f: A -> B, g: B -> A, default: B, a: A) {
    schroeder_bernstein_chain(f, g, a) implies
        schroeder_bernstein_map(f, g, default, a) = f(a)
}

/// Outside the chain set, the Schroeder-Bernstein map agrees with the partial
/// inverse of `g`.
theorem schroeder_bernstein_map_of_not_chain[A, B](f: A -> B, g: B -> A, default: B, a: A) {
    not schroeder_bernstein_chain(f, g, a) implies
        schroeder_bernstein_map(f, g, default, a) = injection_inverse(g, default, a)
}

/// Injectivity on the chain part follows from injectivity of `f`.
theorem schroeder_bernstein_map_eq_chain_chain[A, B](f: A -> B, g: B -> A,
    default: B, a1: A, a2: A) {
    is_injective_fn(f) and schroeder_bernstein_chain(f, g, a1) and
        schroeder_bernstein_chain(f, g, a2) and
        schroeder_bernstein_map(f, g, default, a1) =
        schroeder_bernstein_map(f, g, default, a2)
        implies a1 = a2
} by {
    if is_injective_fn(f) and schroeder_bernstein_chain(f, g, a1) and
        schroeder_bernstein_chain(f, g, a2) and
        schroeder_bernstein_map(f, g, default, a1) =
        schroeder_bernstein_map(f, g, default, a2) {
        schroeder_bernstein_map_of_chain(f, g, default, a1)
        schroeder_bernstein_map_of_chain(f, g, default, a2)
        schroeder_bernstein_map(f, g, default, a1) = f(a1)
        schroeder_bernstein_map(f, g, default, a2) = f(a2)
        f(a1) = f(a2)
        injective_fn_eq(f, a1, a2)
        a1 = a2
    }
}

/// If a chain point has the same image as another point, then the other point
/// is also in the chain set.
theorem schroeder_bernstein_map_eq_chain_imp_chain[A, B](f: A -> B, g: B -> A,
    default: B, a1: A, a2: A) {
    schroeder_bernstein_chain(f, g, a1) and
        schroeder_bernstein_map(f, g, default, a1) =
        schroeder_bernstein_map(f, g, default, a2)
        implies schroeder_bernstein_chain(f, g, a2)
} by {
    if schroeder_bernstein_chain(f, g, a1) and
        schroeder_bernstein_map(f, g, default, a1) =
        schroeder_bernstein_map(f, g, default, a2) {
        if not schroeder_bernstein_chain(f, g, a2) {
            schroeder_bernstein_map_of_chain(f, g, default, a1)
            schroeder_bernstein_map_of_not_chain(f, g, default, a2)
            schroeder_bernstein_map(f, g, default, a1) = f(a1)
            schroeder_bernstein_map(f, g, default, a2) = injection_inverse(g, default, a2)
            f(a1) = injection_inverse(g, default, a2)
            not_chain_imp_image_contains(f, g, a2)
            image_contains(g, a2)
            injection_inverse_maps_to_target(g, default, a2)
            g(injection_inverse(g, default, a2)) = a2
            g(f(a1)) = a2
            schroeder_bernstein_chain_forward(f, g, a1)
            schroeder_bernstein_chain(f, g, g(f(a1)))
            schroeder_bernstein_chain(f, g, a2)
        }
    }
}

/// Outside the chain part, injectivity follows from the partial inverse of `g`.
theorem schroeder_bernstein_map_eq_not_chain_not_chain[A, B](f: A -> B, g: B -> A,
    default: B, a1: A, a2: A) {
    not schroeder_bernstein_chain(f, g, a1) and not schroeder_bernstein_chain(f, g, a2) and
        schroeder_bernstein_map(f, g, default, a1) =
        schroeder_bernstein_map(f, g, default, a2)
        implies a1 = a2
} by {
    if not schroeder_bernstein_chain(f, g, a1) and not schroeder_bernstein_chain(f, g, a2) and
        schroeder_bernstein_map(f, g, default, a1) =
        schroeder_bernstein_map(f, g, default, a2) {
        schroeder_bernstein_map_of_not_chain(f, g, default, a1)
        schroeder_bernstein_map_of_not_chain(f, g, default, a2)
        schroeder_bernstein_map(f, g, default, a1) = injection_inverse(g, default, a1)
        schroeder_bernstein_map(f, g, default, a2) = injection_inverse(g, default, a2)
        injection_inverse(g, default, a1) = injection_inverse(g, default, a2)
        not_chain_imp_image_contains(f, g, a1)
        not_chain_imp_image_contains(f, g, a2)
        image_contains(g, a1)
        image_contains(g, a2)
        injection_inverse_maps_to_target(g, default, a1)
        injection_inverse_maps_to_target(g, default, a2)
        g(injection_inverse(g, default, a1)) = a1
        g(injection_inverse(g, default, a2)) = a2
        a1 = a2
    }
}

/// The Schroeder-Bernstein map is injective.
theorem schroeder_bernstein_map_injective[A, B](f: A -> B, g: B -> A, default: B) {
    is_injective_fn(f) and is_injective_fn(g) implies
        is_injective_fn(schroeder_bernstein_map(f, g, default))
} by {
    if is_injective_fn(f) and is_injective_fn(g) {
        let h: A -> B = schroeder_bernstein_map(f, g, default)
        forall(a1: A, a2: A) {
            if h(a1) = h(a2) {
                schroeder_bernstein_map(f, g, default, a1) = h(a1)
                schroeder_bernstein_map(f, g, default, a2) = h(a2)
                schroeder_bernstein_map(f, g, default, a1) =
                    schroeder_bernstein_map(f, g, default, a2)
                if schroeder_bernstein_chain(f, g, a1) {
                    schroeder_bernstein_map_eq_chain_imp_chain(f, g, default, a1, a2)
                    schroeder_bernstein_chain(f, g, a2)
                    schroeder_bernstein_map_eq_chain_chain(f, g, default, a1, a2)
                    a1 = a2
                } else {
                    if schroeder_bernstein_chain(f, g, a2) {
                        schroeder_bernstein_map(f, g, default, a2) =
                            schroeder_bernstein_map(f, g, default, a1)
                        schroeder_bernstein_map_eq_chain_imp_chain(f, g, default, a2, a1)
                        schroeder_bernstein_chain(f, g, a1)
                        schroeder_bernstein_map_eq_chain_chain(f, g, default, a1, a2)
                        a1 = a2
                    } else {
                        schroeder_bernstein_map_eq_not_chain_not_chain(f, g, default, a1, a2)
                        a1 = a2
                    }
                }
            }
        }
        is_injective_fn(h) = forall(x: A, y: A) {
            h(x) = h(y) implies x = y
        }
        injective_fn_intro_from_eq(h)
        is_injective_fn(h)
        h = schroeder_bernstein_map(f, g, default)
        function_eq_transport_injective_fn(h, schroeder_bernstein_map(f, g, default))
        is_injective_fn(schroeder_bernstein_map(f, g, default))
    }
}

/// The Schroeder-Bernstein map is surjective.
theorem schroeder_bernstein_map_surjective[A, B](f: A -> B, g: B -> A, default: B) {
    is_injective_fn(g) implies is_surjective_fn(schroeder_bernstein_map(f, g, default))
} by {
    if is_injective_fn(g) {
        forall(y: B) {
            if schroeder_bernstein_chain(f, g, g(y)) {
                schroeder_bernstein_chain_backward_image(f, g, y)
                let a: A satisfy {
                    schroeder_bernstein_chain(f, g, a) and f(a) = y
                }
                schroeder_bernstein_map_of_chain(f, g, default, a)
                schroeder_bernstein_map(f, g, default, a) = f(a)
                schroeder_bernstein_map(f, g, default, a) = y
                exists(preimage: A) {
                    schroeder_bernstein_map(f, g, default, preimage) = y
                }
            } else {
                schroeder_bernstein_map_of_not_chain(f, g, default, g(y))
                schroeder_bernstein_map(f, g, default, g(y)) = injection_inverse(g, default, g(y))
                injection_inverse_apply_image(g, default, y)
                injection_inverse(g, default, g(y)) = y
                schroeder_bernstein_map(f, g, default, g(y)) = y
                exists(preimage: A) {
                    schroeder_bernstein_map(f, g, default, preimage) = y
                }
            }
        }
        is_surjective_fn(schroeder_bernstein_map(f, g, default))
    }
}

/// Schroeder-Bernstein with an explicit default element for the codomain.
theorem schroeder_bernstein_with_default[A, B](f: A -> B, g: B -> A, default: B) {
    is_injective_fn(f) and is_injective_fn(g) implies exists(h: A -> B) {
        is_bijection_fn(h)
    }
} by {
    if is_injective_fn(f) and is_injective_fn(g) {
        schroeder_bernstein_map_injective(f, g, default)
        is_injective_fn(schroeder_bernstein_map(f, g, default))
        schroeder_bernstein_map_surjective(f, g, default)
        is_surjective_fn(schroeder_bernstein_map(f, g, default))
        is_bijection_fn(schroeder_bernstein_map(f, g, default))
        exists(h: A -> B) {
            is_bijection_fn(h)
        }
    }
}

/// AixBot-generated Freek Top 100 #25: Schroeder-Bernstein theorem.
theorem top100_025_schroeder_bernstein[A, B](f: A -> B, g: B -> A) {
    is_injective_fn(f) and is_injective_fn(g) implies exists(h: A -> B) {
        is_bijection_fn(h)
    }
} by {
    if is_injective_fn(f) and is_injective_fn(g) {
        if exists(default: B) { has_element_witness(default) } {
            let default: B satisfy {
                has_element_witness(default)
            }
            schroeder_bernstein_with_default(f, g, default)
            exists(h: A -> B) {
                is_bijection_fn(h)
            }
        } else {
            function_to_empty_is_surjective(f)
            is_surjective_fn(f)
            is_bijection_fn(f)
            exists(h: A -> B) {
                is_bijection_fn(h)
            }
        }
    }
}
