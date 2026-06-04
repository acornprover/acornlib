from nat import Nat
from nat import exists_infinite_primes

/// AixBot-generated Freek Top 100 #11: there are infinitely many primes.
theorem top100_011_infinitude_of_primes(n: Nat) {
    exists(p: Nat) {
        n < p and p.is_prime
    }
} by {
    exists_infinite_primes(n)
}
