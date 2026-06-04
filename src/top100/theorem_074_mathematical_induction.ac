from nat import Nat, alt_induction

/// Freek Top 100 #74: the principle of mathematical induction.
theorem top100_074_mathematical_induction(p: Nat -> Bool) {
    (p(Nat.0) and forall(n: Nat) { p(n) implies p(n.suc) })
    implies
    forall(n: Nat) { p(n) }
} by {
    alt_induction(p)
}
