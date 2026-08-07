from top100.theorem_063_cantor_theorem import top100_063_cantor_theorem
from data.basic.functions import bijection_fn_is_surjective, is_bijection_fn,
    is_right_inverse_fn, is_two_sided_inverse_fn,
    right_inverse_fn_imp_surjective_fn, two_sided_inverse_fn_imp_bijection_fn
from data.basic.set import Set

/// There is no bijective function from a type to its powerset.
theorem top100_063_no_bijection_fn_to_powerset[T](f: T -> Set[T]) {
    not is_bijection_fn[T, Set[T]](f)
} by {
    if is_bijection_fn[T, Set[T]](f) {
        bijection_fn_is_surjective[T, Set[T]](f)
        top100_063_cantor_theorem[T](f)
        false
    }
}

/// A map into a powerset has no right inverse.
theorem top100_063_no_right_inverse_to_powerset[T](f: T -> Set[T], g: Set[T] -> T) {
    not is_right_inverse_fn[T, Set[T]](f, g)
} by {
    if is_right_inverse_fn[T, Set[T]](f, g) {
        right_inverse_fn_imp_surjective_fn[T, Set[T]](f, g)
        top100_063_cantor_theorem[T](f)
        false
    }
}

/// There is no two-sided inverse between a type and its powerset in this direction.
theorem top100_063_no_two_sided_inverse_to_powerset[T](f: T -> Set[T], g: Set[T] -> T) {
    not is_two_sided_inverse_fn[T, Set[T]](f, g)
} by {
    if is_two_sided_inverse_fn[T, Set[T]](f, g) {
        two_sided_inverse_fn_imp_bijection_fn[T, Set[T]](f, g)
        top100_063_no_bijection_fn_to_powerset[T](f)
        false
    }
}
