from finite_set import FiniteSet, finite_set_inclusion_exclusion, fs_intersection, fs_union
from nat import Nat

/// Freek Top 100 #96: inclusion-exclusion for two finite sets.
theorem top100_096_inclusion_exclusion[T](s: FiniteSet[T], t: FiniteSet[T],
    n_s: Nat, n_t: Nat, n_inter: Nat) {
    s.cardinality_is(n_s) and
    t.cardinality_is(n_t) and
    fs_intersection[T](s, t).cardinality_is(n_inter)
    implies fs_union[T](s, t).cardinality_is(n_s + n_t - n_inter)
} by {
    finite_set_inclusion_exclusion[T](s, t, n_s, n_t, n_inter)
}
