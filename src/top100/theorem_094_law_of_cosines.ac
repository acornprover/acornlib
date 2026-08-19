from geometry import Point2, point2_law_of_cosines_angle, point2_norm, cos_angle
from real import Real

/// Freek Top 100 #94: the law of cosines in the real coordinate plane, in
/// angle-and-side form at vertex `a` with `u = b - a` and `v = c - a`:
/// `|b-c|² = |b-a|² + |c-a|² − 2|b-a||c-a| (∠BAC).cos`, where
/// `(∠BAC).cos = cos_angle(u, v) = u·v / (|u| |v|)`.
///
/// The hypothesis `b != a and c != a` says the two sides meeting at `a` are
/// nonzero, so the angle is defined.
theorem top100_094_law_of_cosines(a: Point2[Real], b: Point2[Real], c: Point2[Real]) {
    b != a and c != a implies
    b.dist_sq(c) =
        a.dist_sq(b) + a.dist_sq(c) -
        (Real.1 + Real.1) * (point2_norm(b.sub(a)) * point2_norm(c.sub(a)) *
            cos_angle(b.sub(a), c.sub(a)))
} by {
    point2_law_of_cosines_angle(a, b, c)
}
