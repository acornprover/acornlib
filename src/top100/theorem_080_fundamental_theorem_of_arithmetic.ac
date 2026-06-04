from list import List
from list import is_permutation
from list import product
from nat import Nat
from number_theory import all_prime, prime_factorisation_exists,
    prime_factorisation_unique

/// AixBot-generated Freek Top 100 #80: existence and uniqueness of prime factorisation.
theorem top100_080_fundamental_theorem_of_arithmetic(n: Nat) {
    Nat.1 <= n implies exists(factors: List[Nat]) {
        product[Nat](factors) = n and all_prime(factors) and
        forall(other: List[Nat]) {
            product[Nat](other) = n and all_prime(other) implies
            is_permutation[Nat](factors, other)
        }
    }
} by {
    if Nat.1 <= n {
        prime_factorisation_exists(n)
        let factors: List[Nat] satisfy {
            product[Nat](factors) = n and all_prime(factors)
        }
        forall(other: List[Nat]) {
            if product[Nat](other) = n and all_prime(other) {
                product[Nat](factors) = product[Nat](other)
                prime_factorisation_unique(factors, other)
                is_permutation[Nat](factors, other)
            }
        }
        exists(fs: List[Nat]) {
            fs = factors and product[Nat](fs) = n and all_prime(fs) and
            forall(other: List[Nat]) {
                product[Nat](other) = n and all_prime(other) implies
                is_permutation[Nat](fs, other)
            }
        }
    }
}
