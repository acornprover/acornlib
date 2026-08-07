from nat import Nat
from rat import Rat
from data.basic.functions import is_injective_fn, is_surjective_fn
from rat import rational_numbers_have_countably_infinite_enumeration

/// AixBot-generated Freek Top 100 #3: the rational numbers are denumerable.
theorem top100_003_denumerability_of_rationals {
    exists(enumeration: Nat -> Rat) {
        is_surjective_fn(enumeration)
    } and exists(encoding: Rat -> Nat) {
        is_injective_fn(encoding)
    } and exists(embedding: Nat -> Rat) {
        is_injective_fn(embedding)
    }
} by {
    rational_numbers_have_countably_infinite_enumeration
}
