from list import List, map
from list import is_permutation, permutation_preserves_product
from list import product
from nat import Nat
from number_theory import congr_mod_mul, congr_mod_refl, congr_mod_symm
from number_theory import cancel_coprime
from number_theory import coprime_residues, coprime_residues_length, mul_mod_fn,
    mul_mod_residues, mul_mod_residues_is_permutation, product_coprime_residues_coprime,
    product_map_cons, product_map_scalar, scalar_mul_fn

/// Helper: multiplication modulo `n` preserves the product modulo `n`.
theorem product_map_mul_mod_congr(n: Nat, a: Nat, l: List[Nat]) {
    product[Nat](map[Nat, Nat](l, mul_mod_fn(n, a))).congr_mod(
        product[Nat](map[Nat, Nat](l, scalar_mul_fn(a))),
        n
    )
} by {
    define p(x: List[Nat]) -> Bool {
        product[Nat](map[Nat, Nat](x, mul_mod_fn(n, a))).congr_mod(
            product[Nat](map[Nat, Nat](x, scalar_mul_fn(a))),
            n
        )
    }
    map[Nat, Nat](List.nil[Nat], mul_mod_fn(n, a)) = List.nil[Nat]
    map[Nat, Nat](List.nil[Nat], scalar_mul_fn(a)) = List.nil[Nat]
    product[Nat](List.nil[Nat]) = Nat.1
    congr_mod_refl(Nat.1, n)
    p(List.nil[Nat])
    forall(head: Nat, tail: List[Nat]) {
        if p(tail) {
            mul_mod_fn(n, a)(head).congr_mod(scalar_mul_fn(a)(head), n)
            product[Nat](map[Nat, Nat](tail, mul_mod_fn(n, a))).congr_mod(
                product[Nat](map[Nat, Nat](tail, scalar_mul_fn(a))),
                n
            )
            congr_mod_mul(
                mul_mod_fn(n, a)(head),
                scalar_mul_fn(a)(head),
                product[Nat](map[Nat, Nat](tail, mul_mod_fn(n, a))),
                product[Nat](map[Nat, Nat](tail, scalar_mul_fn(a))),
                n
            )
            (
                mul_mod_fn(n, a)(head) *
                product[Nat](map[Nat, Nat](tail, mul_mod_fn(n, a)))
            ).congr_mod(
                scalar_mul_fn(a)(head) *
                product[Nat](map[Nat, Nat](tail, scalar_mul_fn(a))),
                n
            )
            product_map_cons(mul_mod_fn(n, a), head, tail)
            product_map_cons(scalar_mul_fn(a), head, tail)
            p(List.cons(head, tail))
        }
    }
    List.induction(function(x: List[Nat]) { p(x) })
    forall(x: List[Nat]) { p(x) }
    p(l)
}

/// AixBot-generated Freek Top 100 #10: Euler's generalization of Fermat's little theorem.
theorem top100_010_eulers_generalization_of_fermats_little_theorem(n: Nat, a: Nat) {
    n != Nat.0 and a.coprime(n) implies a.pow(n.totient).congr_mod(Nat.1, n)
} by {
    if n != Nat.0 and a.coprime(n) {
        let r: List[Nat] = coprime_residues(n)
        let pr: Nat = product[Nat](r)
        product_coprime_residues_coprime(n)
        pr.coprime(n)
        coprime_residues_length(n)
        r.length = n.totient
        mul_mod_residues_is_permutation(n, a)
        is_permutation[Nat](mul_mod_residues(n, a), r)
        permutation_preserves_product(mul_mod_residues(n, a), r)
        product[Nat](mul_mod_residues(n, a)) = pr
        mul_mod_residues(n, a) = map[Nat, Nat](r, mul_mod_fn(n, a))
        product[Nat](map[Nat, Nat](r, mul_mod_fn(n, a))) = pr
        product_map_mul_mod_congr(n, a, r)
        product[Nat](map[Nat, Nat](r, mul_mod_fn(n, a))).congr_mod(
            product[Nat](map[Nat, Nat](r, scalar_mul_fn(a))),
            n
        )
        pr.congr_mod(product[Nat](map[Nat, Nat](r, scalar_mul_fn(a))), n)
        product_map_scalar(a, r)
        product[Nat](map[Nat, Nat](r, scalar_mul_fn(a))) = a.pow(r.length) * pr
        pr.congr_mod(a.pow(r.length) * pr, n)
        congr_mod_symm(pr, a.pow(r.length) * pr, n)
        (a.pow(r.length) * pr).congr_mod(pr, n)
        a.pow(r.length) * pr = pr * a.pow(r.length)
        pr = pr * Nat.1
        (pr * a.pow(r.length)).congr_mod(pr * Nat.1, n)
        r.length = n.totient
        (pr * a.pow(n.totient)).congr_mod(pr * Nat.1, n)
        cancel_coprime(pr, n, a.pow(n.totient), Nat.1)
        a.pow(n.totient).congr_mod(Nat.1, n)
    }
}
