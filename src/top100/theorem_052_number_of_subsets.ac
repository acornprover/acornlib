from finite_set import FiniteSet, finite_powerset_from_list,
    finite_powerset_from_list_cardinality, finite_powerset_from_list_exact
from list import List
from nat import Nat

/// AixBot-generated Freek Top 100 #52: a finite set with n elements has 2^n subsets.
theorem top100_052_number_of_subsets_of_a_finite_set[A](items: List[A]) {
    items.is_unique implies
    forall(s: FiniteSet[A]) {
        finite_powerset_from_list[A](items).contains(s) = s.subset_eq(FiniteSet.from_list[A](items))
    } and
    finite_powerset_from_list[A](items).cardinality_is(Nat.2.pow(items.length))
} by {
    finite_powerset_from_list_exact[A](items)
    finite_powerset_from_list_cardinality[A](items)
}
