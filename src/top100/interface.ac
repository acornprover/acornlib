from comm_ring import CommRing, binomial_term
from finite_group import FiniteGroup, FiniteSubgroup
from finite_set import FiniteSet, finite_powerset_from_list, fs_intersection, fs_union
from data.basic.functions import is_bijection_fn, is_injective_fn, is_surjective_fn
from geometry import Point2, heron_linear_product
from int import Int
from data.int.int_digits import int_digit_sum, int_read_digits_acc
from list import List, map, partial, sum
from binary_words import successful_ballot_words
from list import is_permutation
from list import product
from combinatorics import binom
from number_theory import congr_mod_refl
from number_theory import coprime_one_left
from number_theory import all_prime
from number_theory import gcd_step_n
from number_theory import count_coprime_to
from pair import Pair
from rat import Rat
from real import Real
from real import finite_dot, finite_square_sum
from real import harmonic
from real import converges
from real import nat_to_real
from algebra.ring.ring import Ring
from nat import Nat, from_nat
from data.basic.set import Set

/// Freek Top 100 #1: irrationality of the square root of 2.
theorem top100_001_sqrt2_irrational_core(r: Rat) {
    r * r != Rat.2
}

/// Freek Top 100 #3: the rational numbers are denumerable.
theorem top100_003_denumerability_of_rationals {
    exists(enumeration: Nat -> Rat) {
        is_surjective_fn(enumeration)
    } and exists(encoding: Rat -> Nat) {
        is_injective_fn(encoding)
    } and exists(embedding: Nat -> Rat) {
        is_injective_fn(embedding)
    }
}

/// Freek Top 100 #10: Euler's generalization of Fermat's little theorem.
theorem top100_010_eulers_generalization_of_fermats_little_theorem(n: Nat, a: Nat) {
    n != Nat.0 and a.coprime(n) implies a.pow(n.totient).congr_mod(Nat.1, n)
}

/// Freek Top 100 #11: there are infinitely many primes.
theorem top100_011_infinitude_of_primes(n: Nat) {
    exists(p: Nat) {
        n < p and p.is_prime
    }
}

/// Freek Top 100 #25: Schroeder-Bernstein theorem.
theorem top100_025_schroeder_bernstein[A, B](f: A -> B, g: B -> A) {
    is_injective_fn(f) and is_injective_fn(g) implies exists(h: A -> B) {
        is_bijection_fn(h)
    }
}

/// Freek Top 100 #30: the Ballot Problem, in binomial-difference form.
theorem top100_030_the_ballot_problem(positives: Nat, negatives: Nat) {
    negatives <= positives implies
        successful_ballot_words(positives.suc, negatives.suc).length +
            (positives + negatives.suc).binom(positives.suc) =
            (positives + negatives.suc).binom(positives)
}

/// Freek Top 100 #34: the harmonic series does not converge.
theorem top100_034_divergence_of_harmonic_series {
    not converges(partial[Real](harmonic))
}

/// Freek Top 100 #44: the binomial theorem for commutative rings.
theorem top100_044_binomial_theorem[R: CommRing](a: R, b: R, n: Nat) {
    (a + b).pow(n) = partial(binomial_term[R](a, b, n), n.suc)
}

/// Freek Top 100 #51: Wilson's theorem.
theorem top100_051_wilsons_theorem(p: Nat) {
    p != Nat.1 implies
        (p.is_prime = (p - Nat.1).factorial.congr_mod(p - Nat.1, p))
}

/// Freek Top 100 #58: the formula for the number of combinations.
theorem top100_058_combinations_formula(n: Nat, k: Nat) {
    k <= n implies n.binom(k) * k.factorial * (n - k).factorial = n.factorial
}

/// Freek Top 100 #60: Bezout's identity for integer gcds.
theorem top100_060_bezout_identity(a: Int, b: Int) {
    exists(x: Int, y: Int) {
        a.gcd(b) = a * x + b * y
    }
}

/// Freek Top 100 #60: Bezout's theorem for integer gcds.
theorem top100_060_bezouts_theorem(a: Int, b: Int) {
    exists(d: Int) {
        not d.is_negative and
        d.divides(a) and
        d.divides(b) and
        exists(x: Int, y: Int) {
            d = a * x + b * y
        }
    }
}

/// Freek Top 100 #63: Cantor's theorem.
theorem top100_063_cantor_theorem[T](f: T -> Set[T]) {
    not is_surjective_fn[T, Set[T]](f)
}

/// Freek Top 100 #66: the finite sum formula for a geometric series.
theorem top100_066_geometric_series[R: Ring](a: R, n: Nat) {
    (a + -R.1) * sum(map(n.range, a.pow)) = a.pow(n) + -R.1
}

/// Freek Top 100 #68: the sum of an arithmetic series.
theorem top100_068_arithmetic_series(n: Nat) {
    partial[Real](nat_to_real, n) = nat_to_real(n) * nat_to_real(n - Nat.1) * Real.one_half
}

/// Freek Top 100 #69: the Euclidean algorithm computes greatest common divisors.
theorem top100_069_greatest_common_divisor_algorithm(a: Nat, b: Nat) {
    exists(n: Nat) {
        gcd_step_n(Pair.new(a, b), n).second = Nat.0 and
        gcd_step_n(Pair.new(a, b), n).first = a.gcd(b) and
        a.gcd(b).divides(a) and
        a.gcd(b).divides(b) and
        forall(d: Nat) {
            d.divides(a) and d.divides(b) implies d.divides(a.gcd(b))
        }
    }
}

/// Freek Top 100 #71: Lagrange's theorem for finite groups.
theorem top100_071_lagrange_theorem[G: FiniteGroup](s: FiniteSubgroup[G]) {
    s.order.divides(G.order)
}

/// Freek Top 100 #74: the principle of mathematical induction.
theorem top100_074_mathematical_induction(p: Nat -> Bool) {
    (p(Nat.0) and forall(n: Nat) { p(n) implies p(n.suc) })
    implies
    forall(n: Nat) { p(n) }
}

/// Freek Top 100 #78: the finite real Cauchy-Schwarz inequality.
theorem top100_078_cauchy_schwarz(a: Nat -> Real, b: Nat -> Real, n: Nat) {
    finite_dot(a, b, n) * finite_dot(a, b, n) <=
        finite_square_sum(a, n) * finite_square_sum(b, n)
}

/// Freek Top 100 #80: existence and uniqueness of prime factorisation.
theorem top100_080_fundamental_theorem_of_arithmetic(n: Nat) {
    Nat.1 <= n implies exists(factors: List[Nat]) {
        product[Nat](factors) = n and all_prime(factors) and
        forall(other: List[Nat]) {
            product[Nat](other) = n and all_prime(other) implies
            is_permutation[Nat](factors, other)
        }
    }
}

/// Freek Top 100 #85: divisibility by 3 is preserved by taking the digit sum.
theorem top100_085_divisibility_by_3_rule(digits: List[Int]) {
    Int.3.divides(int_read_digits_acc(digits, Int.0)) =
    Int.3.divides(int_digit_sum(digits))
}

/// Componentwise addition on real coordinate pairs.
define r2_add(a: Pair[Real, Real], b: Pair[Real, Real]) -> Pair[Real, Real] {
    Pair.new(a.first + b.first, a.second + b.second)
}

/// Componentwise negation on real coordinate pairs.
define r2_neg(a: Pair[Real, Real]) -> Pair[Real, Real] {
    Pair.new(-a.first, -a.second)
}

/// Componentwise subtraction on real coordinate pairs.
define r2_sub(a: Pair[Real, Real], b: Pair[Real, Real]) -> Pair[Real, Real] {
    Pair.new(a.first - b.first, a.second - b.second)
}

/// The Euclidean dot product on real coordinate pairs.
define r2_dot(a: Pair[Real, Real], b: Pair[Real, Real]) -> Real {
    a.first * b.first + a.second * b.second
}

/// The square of the Euclidean norm on real coordinate pairs.
define r2_norm_sq(a: Pair[Real, Real]) -> Real {
    r2_dot(a, a)
}

/// True if two real coordinate pairs are orthogonal.
define r2_orthogonal(a: Pair[Real, Real], b: Pair[Real, Real]) -> Bool {
    r2_dot(a, b) = Real.0
}

/// Freek Top 100 #4: the Pythagorean theorem in the real coordinate plane.
theorem top100_004_pythagorean_theorem(
    a: Pair[Real, Real], b: Pair[Real, Real], c: Pair[Real, Real]
) {
    r2_orthogonal(r2_sub(a, b), r2_sub(c, b)) implies
    r2_norm_sq(r2_sub(c, a)) = r2_norm_sq(r2_sub(b, a)) + r2_norm_sq(r2_sub(c, b))
}

/// The reciprocal of the kth triangular number, indexed from zero.
define triangular_term(k: Nat) -> Real {
    Real.1 / ((from_nat[Real](k.suc) * from_nat[Real](k.suc.suc)) / (Real.1 + Real.1))
}

/// The finite sum of reciprocal triangular numbers up to n.
define triangular_reciprocal_sum(n: Nat) -> Real {
    match n {
        Nat.zero { Real.0 }
        Nat.suc(m) { triangular_reciprocal_sum(m) + triangular_term(m) }
    }
}

/// The finite sum of reciprocal triangular numbers has the Top 100 closed form.
theorem top100_042_triangular_reciprocal_sum(n: Nat) {
    triangular_reciprocal_sum(n) = ((Real.1 + Real.1) * from_nat[Real](n)) / from_nat[Real](n.suc)
}

/// AixBot-generated Freek Top 100 #52: a finite set with n elements has 2^n subsets.
theorem top100_052_number_of_subsets_of_a_finite_set[A](items: List[A]) {
    items.is_unique implies
    forall(s: FiniteSet[A]) {
        finite_powerset_from_list[A](items).contains(s) = s.subset_eq(FiniteSet.from_list[A](items))
    } and
    finite_powerset_from_list[A](items).cardinality_is(Nat.2.pow(items.length))
}

/// The four land masses in the Konigsberg bridge graph.
inductive KonigsbergVertex {
    /// The land mass labelled `0` in the classical finite graph statement.
    v0
    /// The land mass labelled `1` in the classical finite graph statement.
    v1
    /// The land mass labelled `2` in the classical finite graph statement.
    v2
    /// The land mass labelled `3` in the classical finite graph statement.
    v3
}

/// The seven bridges in the Konigsberg bridge graph.
inductive KonigsbergBridge {
    /// The bridge whose endpoints are `v0` and `v1`.
    e10
    /// The bridge whose endpoints are `v0` and `v2`.
    e20
    /// The bridge whose endpoints are `v0` and `v3`.
    e30
    /// The first bridge whose endpoints are `v1` and `v2`.
    e40
    /// The second bridge whose endpoints are `v1` and `v2`.
    e50
    /// The first bridge whose endpoints are `v2` and `v3`.
    e60
    /// The second bridge whose endpoints are `v2` and `v3`.
    e70
}

/// A bridge crossed in one of its two possible directions.
inductive DirectedKonigsbergBridge {
    /// Bridge `e10`, crossed from `v0` to `v1`.
    e10_01
    /// Bridge `e10`, crossed from `v1` to `v0`.
    e10_10
    /// Bridge `e20`, crossed from `v0` to `v2`.
    e20_02
    /// Bridge `e20`, crossed from `v2` to `v0`.
    e20_20
    /// Bridge `e30`, crossed from `v0` to `v3`.
    e30_03
    /// Bridge `e30`, crossed from `v3` to `v0`.
    e30_30
    /// Bridge `e40`, crossed from `v1` to `v2`.
    e40_12
    /// Bridge `e40`, crossed from `v2` to `v1`.
    e40_21
    /// Bridge `e50`, crossed from `v1` to `v2`.
    e50_12
    /// Bridge `e50`, crossed from `v2` to `v1`.
    e50_21
    /// Bridge `e60`, crossed from `v2` to `v3`.
    e60_23
    /// Bridge `e60`, crossed from `v3` to `v2`.
    e60_32
    /// Bridge `e70`, crossed from `v2` to `v3`.
    e70_23
    /// Bridge `e70`, crossed from `v3` to `v2`.
    e70_32
}

/// The underlying bridge of a directed crossing.
define directed_bridge(step: DirectedKonigsbergBridge) -> KonigsbergBridge {
    match step {
        DirectedKonigsbergBridge.e10_01 { KonigsbergBridge.e10 }
        DirectedKonigsbergBridge.e10_10 { KonigsbergBridge.e10 }
        DirectedKonigsbergBridge.e20_02 { KonigsbergBridge.e20 }
        DirectedKonigsbergBridge.e20_20 { KonigsbergBridge.e20 }
        DirectedKonigsbergBridge.e30_03 { KonigsbergBridge.e30 }
        DirectedKonigsbergBridge.e30_30 { KonigsbergBridge.e30 }
        DirectedKonigsbergBridge.e40_12 { KonigsbergBridge.e40 }
        DirectedKonigsbergBridge.e40_21 { KonigsbergBridge.e40 }
        DirectedKonigsbergBridge.e50_12 { KonigsbergBridge.e50 }
        DirectedKonigsbergBridge.e50_21 { KonigsbergBridge.e50 }
        DirectedKonigsbergBridge.e60_23 { KonigsbergBridge.e60 }
        DirectedKonigsbergBridge.e60_32 { KonigsbergBridge.e60 }
        DirectedKonigsbergBridge.e70_23 { KonigsbergBridge.e70 }
        DirectedKonigsbergBridge.e70_32 { KonigsbergBridge.e70 }
    }
}

/// The source land mass of a directed crossing.
define directed_source(step: DirectedKonigsbergBridge) -> KonigsbergVertex {
    match step {
        DirectedKonigsbergBridge.e10_01 { KonigsbergVertex.v0 }
        DirectedKonigsbergBridge.e10_10 { KonigsbergVertex.v1 }
        DirectedKonigsbergBridge.e20_02 { KonigsbergVertex.v0 }
        DirectedKonigsbergBridge.e20_20 { KonigsbergVertex.v2 }
        DirectedKonigsbergBridge.e30_03 { KonigsbergVertex.v0 }
        DirectedKonigsbergBridge.e30_30 { KonigsbergVertex.v3 }
        DirectedKonigsbergBridge.e40_12 { KonigsbergVertex.v1 }
        DirectedKonigsbergBridge.e40_21 { KonigsbergVertex.v2 }
        DirectedKonigsbergBridge.e50_12 { KonigsbergVertex.v1 }
        DirectedKonigsbergBridge.e50_21 { KonigsbergVertex.v2 }
        DirectedKonigsbergBridge.e60_23 { KonigsbergVertex.v2 }
        DirectedKonigsbergBridge.e60_32 { KonigsbergVertex.v3 }
        DirectedKonigsbergBridge.e70_23 { KonigsbergVertex.v2 }
        DirectedKonigsbergBridge.e70_32 { KonigsbergVertex.v3 }
    }
}

/// The target land mass of a directed crossing.
define directed_target(step: DirectedKonigsbergBridge) -> KonigsbergVertex {
    match step {
        DirectedKonigsbergBridge.e10_01 { KonigsbergVertex.v1 }
        DirectedKonigsbergBridge.e10_10 { KonigsbergVertex.v0 }
        DirectedKonigsbergBridge.e20_02 { KonigsbergVertex.v2 }
        DirectedKonigsbergBridge.e20_20 { KonigsbergVertex.v0 }
        DirectedKonigsbergBridge.e30_03 { KonigsbergVertex.v3 }
        DirectedKonigsbergBridge.e30_30 { KonigsbergVertex.v0 }
        DirectedKonigsbergBridge.e40_12 { KonigsbergVertex.v2 }
        DirectedKonigsbergBridge.e40_21 { KonigsbergVertex.v1 }
        DirectedKonigsbergBridge.e50_12 { KonigsbergVertex.v2 }
        DirectedKonigsbergBridge.e50_21 { KonigsbergVertex.v1 }
        DirectedKonigsbergBridge.e60_23 { KonigsbergVertex.v3 }
        DirectedKonigsbergBridge.e60_32 { KonigsbergVertex.v2 }
        DirectedKonigsbergBridge.e70_23 { KonigsbergVertex.v3 }
        DirectedKonigsbergBridge.e70_32 { KonigsbergVertex.v2 }
    }
}

/// True if a directed list of bridge crossings forms a walk from `start` to `end`.
define directed_konigsberg_walk(
    start: KonigsbergVertex, steps: List[DirectedKonigsbergBridge], end: KonigsbergVertex
) -> Bool {
    match steps {
        List.nil {
            start = end
        }
        List.cons(step, tail) {
            directed_source(step) = start and directed_konigsberg_walk(directed_target(step), tail, end)
        }
    }
}

/// The number of times an underlying bridge is used by a directed crossing list.
define directed_bridge_count(steps: List[DirectedKonigsbergBridge], bridge: KonigsbergBridge) -> Nat {
    match steps {
        List.nil { Nat.0 }
        List.cons(step, tail) {
            if directed_bridge(step) = bridge {
                Nat.1 + directed_bridge_count(tail, bridge)
            } else {
                directed_bridge_count(tail, bridge)
            }
        }
    }
}

/// True if every underlying Konigsberg bridge is used exactly once.
define directed_uses_every_bridge_once(steps: List[DirectedKonigsbergBridge]) -> Bool {
    directed_bridge_count(steps, KonigsbergBridge.e10) = Nat.1 and
    directed_bridge_count(steps, KonigsbergBridge.e20) = Nat.1 and
    directed_bridge_count(steps, KonigsbergBridge.e30) = Nat.1 and
    directed_bridge_count(steps, KonigsbergBridge.e40) = Nat.1 and
    directed_bridge_count(steps, KonigsbergBridge.e50) = Nat.1 and
    directed_bridge_count(steps, KonigsbergBridge.e60) = Nat.1 and
    directed_bridge_count(steps, KonigsbergBridge.e70) = Nat.1
}

/// True if a directed list of bridge crossings is an Eulerian trail in the Konigsberg graph.
define directed_konigsberg_eulerian_trail(steps: List[DirectedKonigsbergBridge]) -> Bool {
    directed_uses_every_bridge_once(steps) and
        exists(start: KonigsbergVertex, end: KonigsbergVertex) {
            directed_konigsberg_walk(start, steps, end)
        }
}

/// The Konigsberg bridge graph has no Eulerian trail.
theorem top100_054_konigsberg_bridges(steps: List[DirectedKonigsbergBridge]) {
    not directed_konigsberg_eulerian_trail(steps)
}

/// True when `length` is the Euclidean length of a real coordinate vector.
define point2_real_length_witness(p: Point2[Real], length: Real) -> Bool {
    Real.0 <= length and length * length = p.norm_sq
}

/// Freek Top 100 #55: the products of the two segment lengths of intersecting chords are equal.
theorem top100_055_product_of_segments_of_chords(
    center: Point2[Real], radius_sq: Real,
    q: Point2[Real], r: Point2[Real], s: Point2[Real], t: Point2[Real], x: Point2[Real],
    u: Real, v: Real,
    qx: Real, rx: Real, sx: Real, tx: Real
) {
    center.on_circle(radius_sq, q) and
    center.on_circle(radius_sq, r) and
    center.on_circle(radius_sq, s) and
    center.on_circle(radius_sq, t) and
    Real.0 <= u and u <= Real.1 and
    Real.0 <= v and v <= Real.1 and
    x = q.param_line(r, u) and
    x = s.param_line(t, v) and
    point2_real_length_witness(q.sub(x), qx) and
    point2_real_length_witness(r.sub(x), rx) and
    point2_real_length_witness(s.sub(x), sx) and
    point2_real_length_witness(t.sub(x), tx)
    implies qx * rx = sx * tx
}

/// True when `length` is the Euclidean length of a real coordinate vector.
define point2_real_length_witness_57(p: Point2[Real], length: Real) -> Bool {
    Real.0 <= length and length * length = p.norm_sq
}

/// Freek Top 100 #57: Heron's formula in squared doubled-area form.
theorem top100_057_herons_formula_squared(
    a: Point2[Real], b: Point2[Real], c: Point2[Real],
    side_a: Real, side_b: Real, side_c: Real
) {
    point2_real_length_witness_57(c.sub(b), side_a) and
    point2_real_length_witness_57(c.sub(a), side_b) and
    point2_real_length_witness_57(b.sub(a), side_c)
    implies
    heron_linear_product(side_a, side_b, side_c) =
        (a.triangle_area2(b, c) + a.triangle_area2(b, c)) *
        (a.triangle_area2(b, c) + a.triangle_area2(b, c))
}

/// The coefficient function obtained by dropping the constant coefficient.
define coeff_tail(c: Nat -> Real, i: Nat) -> Real {
    c(i.suc)
}

/// Evaluation of the polynomial determined by the first `n` coefficients.
define coeff_eval(c: Nat -> Real, x: Real, n: Nat) -> Real {
    match n {
        Nat.zero {
            Real.0
        }
        Nat.suc(pred) {
            c(Nat.0) + x * coeff_eval(coeff_tail(c), x, pred)
        }
    }
}

/// The synthetic quotient coefficients for division by `X - a`, using the first `n` coefficients.
define coeff_quotient(c: Nat -> Real, a: Real, n: Nat, i: Nat) -> Real {
    match n {
        Nat.zero {
            Real.0
        }
        Nat.suc(pred) {
            match i {
                Nat.zero {
                    coeff_eval(coeff_tail(c), a, pred)
                }
                Nat.suc(ipred) {
                    coeff_quotient(coeff_tail(c), a, pred, ipred)
                }
            }
        }
    }
}

/// The value at `x` of the product of a coefficient polynomial with `X - a`.
define coeff_mul_x_minus_a(q: Nat -> Real, a: Real, x: Real, n: Nat) -> Real {
    coeff_eval(q, x, n) * (x - a)
}

/// True when a coefficient polynomial is divisible by `X - a`.
define coeff_divisible_by_x_minus_a(c: Nat -> Real, a: Real, n: Nat) -> Bool {
    exists(q: Nat -> Real) {
        forall(x: Real) {
            coeff_eval(c, x, n) = coeff_mul_x_minus_a(q, a, x, n)
        }
    }
}

/// The coefficient-form remainder theorem.
theorem top100_089_remainder_theorem(c: Nat -> Real, a: Real, x: Real, n: Nat) {
    coeff_eval(c, x, n) = coeff_eval(c, a, n) + (x - a) * coeff_eval(coeff_quotient(c, a, n), x, n)
}

/// Freek Top 100 #89: the factor theorem for real coefficient polynomials.
theorem top100_089_factor_theorem(c: Nat -> Real, a: Real, n: Nat) {
    coeff_eval(c, a, n) = Real.0 = coeff_divisible_by_x_minus_a(c, a, n)
}

/// True when `r` is the Euclidean norm of a real coordinate vector.
define point2_real_norm(p: Point2[Real], r: Real) -> Bool {
    Real.0 <= r and r * r = p.norm_sq
}

/// Freek Top 100 #91, specialized to the Euclidean norm on `R^2`.
theorem top100_091_triangle_inequality_r2(
    a: Point2[Real], b: Point2[Real], an: Real, bn: Real, abn: Real
) {
    point2_real_norm(a, an) and point2_real_norm(b, bn) and point2_real_norm(a.add(b), abn)
        implies abn <= an + bn
}

/// Freek Top 100 #96: inclusion-exclusion for two finite sets.
theorem top100_096_inclusion_exclusion[T](s: FiniteSet[T], t: FiniteSet[T],
    n_s: Nat, n_t: Nat, n_inter: Nat) {
    s.cardinality_is(n_s) and
    t.cardinality_is(n_t) and
    fs_intersection[T](s, t).cardinality_is(n_inter)
    implies fs_union[T](s, t).cardinality_is(n_s + n_t - n_inter)
}
