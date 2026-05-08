from real import Real
from pair import Pair, pair_ext

numerals Real

/// Componentwise addition on real coordinate pairs.
define r2_add(a: Pair[Real, Real], b: Pair[Real, Real]) -> Pair[Real, Real] {
    Pair.new(a.first + b.first, a.second + b.second)
}

/// Componentwise negation on real coordinate pairs.
define r2_neg(a: Pair[Real, Real]) -> Pair[Real, Real] {
    Pair.new(-a.first, -a.second)
}

/// Componentwise subtraction on real coordinate pairs.
define r2_sub(a: Pair[Real, Real], b: Pair[Real, Real]) -> Pair[Real, Real] {
    Pair.new(a.first - b.first, a.second - b.second)
}

/// The Euclidean dot product on real coordinate pairs.
define r2_dot(a: Pair[Real, Real], b: Pair[Real, Real]) -> Real {
    a.first * b.first + a.second * b.second
}

/// The square of the Euclidean norm on real coordinate pairs.
define r2_norm_sq(a: Pair[Real, Real]) -> Real {
    r2_dot(a, a)
}

/// True if two real coordinate pairs are orthogonal.
define r2_orthogonal(a: Pair[Real, Real], b: Pair[Real, Real]) -> Bool {
    r2_dot(a, b) = Real.0
}

/// The square of a sum, with the two cross terms written separately.
theorem real_square_add(x: Real, y: Real) {
    (x + y) * (x + y) = x * x + x * y + (x * y + y * y)
} by {
    (x + y) * (x + y) = (x + y) * x + (x + y) * y
    (x + y) * x = x * x + y * x
    (x + y) * y = x * y + y * y
    (x + y) * (x + y) = (x * x + y * x) + (x * y + y * y)
    y * x = x * y
    (x * x + y * x) + (x * y + y * y) = x * x + x * y + (x * y + y * y)
}

/// Move the repeated middle term together in a four-term sum.
theorem real_add_repeated_middle(a: Real, b: Real, c: Real) {
    a + b + (b + c) = a + (b + b) + c
} by {
    a + b + (b + c) = (a + b) + (b + c)
    (a + b) + (b + c) = a + (b + (b + c))
    b + (b + c) = (b + b) + c
    a + (b + (b + c)) = a + ((b + b) + c)
    a + ((b + b) + c) = a + (b + b) + c
}

/// Repeated terms may be paired in either order.
theorem real_double_pair_rearrange(a: Real, b: Real) {
    (a + a) + (b + b) = (a + b) + (a + b)
} by {
    (a + a) + (b + b) = a + (a + (b + b))
    a + (b + (a + b)) = (a + b) + (a + b)
    a + (a + (b + b)) = a + (b + (a + b))
}

/// Move the last term of a four-term sum next to the first.
theorem real_add_four_last_next_to_first(a: Real, b: Real, c: Real, d: Real) {
    a + b + c + d = (a + d) + (b + c)
} by {
    a + b + c + d = ((a + b) + c) + d
    ((a + b) + c) + d = (a + b) + (c + d)
    (a + b) + (c + d) = a + (b + (c + d))
    b + (c + d) = (b + c) + d
    a + ((b + c) + d) = a + (d + (b + c))
    a + (d + (b + c)) = (a + d) + (b + c)
}

/// Pair the first, fourth, third, and sixth terms of a six-term sum.
theorem real_add_six_pair_rearrange(a: Real, b: Real, c: Real, d: Real, e: Real, f: Real) {
    (a + b + c) + (d + e + f) = (a + d) + (c + f) + (b + e)
} by {
    (a + b + c) + (d + e + f) = ((a + b + c) + (d + e)) + f
    ((a + b + c) + (d + e)) + f = (((a + b + c) + d) + e) + f
    real_add_four_last_next_to_first(a, b, c, d)
    a + b + c + d = (a + d) + (b + c)
    (((a + b + c) + d) + e) + f = (((a + d) + (b + c)) + e) + f
    ((a + d) + (b + c)) + e + f = ((a + d) + ((b + c) + e)) + f
    (b + c) + e = c + (b + e)
    ((a + d) + ((b + c) + e)) + f = ((a + d) + (c + (b + e))) + f
    ((a + d) + (c + (b + e))) + f = (a + d) + ((c + (b + e)) + f)
    (c + (b + e)) + f = (c + f) + (b + e)
    (a + d) + ((c + f) + (b + e)) = (a + d) + (c + f) + (b + e)
}

/// Subtraction through an intermediate real number decomposes a difference.
theorem real_sub_through(x: Real, y: Real, z: Real) {
    (y - x) + (z - y) = z - x
} by {
    y - x = y + -x
    z - y = z + -y
    z - x = z + -x
    (y - x) + (z - y) = (y + -x) + (z + -y)
    (y + -x) + (z + -y) = y + -x + z + -y
    real_add_four_last_next_to_first(y, -x, z, -y)
    y + -x + z + -y = (y + -y) + (-x + z)
    y + -y = Real.0
    (y + -y) + (-x + z) = Real.0 + (-x + z)
    Real.0 + (-x + z) = -x + z
    -x + z = z + -x
    (y - x) + (z - y) = z - x
}

/// Reversing a real difference negates it.
theorem real_sub_reverse_neg(x: Real, y: Real) {
    x - y = -(y - x)
} by {
    x - y = x + -y
    y - x = y + -x
    -(y - x) = -(y + -x)
    -(y + -x) = -y + --x
    --x = x
    -y + x = x + -y
}

/// Negation may be moved out of the left side of the dot product.
theorem r2_dot_neg_left(a: Pair[Real, Real], b: Pair[Real, Real]) {
    r2_dot(r2_neg(a), b) = -r2_dot(a, b)
} by {
    r2_neg(a).first = -a.first
    r2_neg(a).second = -a.second
    r2_dot(r2_neg(a), b) = (-a.first) * b.first + (-a.second) * b.second
    (-a.first) * b.first = -(a.first * b.first)
    (-a.second) * b.second = -(a.second * b.second)
    r2_dot(a, b) = a.first * b.first + a.second * b.second
    -(a.first * b.first + a.second * b.second) = -(a.first * b.first) + -(a.second * b.second)
    r2_dot(r2_neg(a), b) = -r2_dot(a, b)
}

/// Reversing the order in a coordinate-pair subtraction negates the vector.
theorem r2_sub_reverse_neg(a: Pair[Real, Real], b: Pair[Real, Real]) {
    r2_sub(a, b) = r2_neg(r2_sub(b, a))
} by {
    let lhs = r2_sub(a, b)
    let rhs = r2_neg(r2_sub(b, a))
    lhs.first = a.first - b.first
    lhs.second = a.second - b.second
    rhs.first = -r2_sub(b, a).first
    r2_sub(b, a).first = b.first - a.first
    rhs.first = -(b.first - a.first)
    real_sub_reverse_neg(a.first, b.first)
    a.first - b.first = -(b.first - a.first)
    rhs.first = lhs.first
    rhs.second = -r2_sub(b, a).second
    r2_sub(b, a).second = b.second - a.second
    rhs.second = -(b.second - a.second)
    real_sub_reverse_neg(a.second, b.second)
    a.second - b.second = -(b.second - a.second)
    rhs.second = lhs.second
    pair_ext(lhs, rhs)
}

/// Orthogonality is unchanged by negating the left vector.
theorem r2_orthogonal_neg_left(a: Pair[Real, Real], b: Pair[Real, Real]) {
    r2_orthogonal(r2_neg(a), b) implies r2_orthogonal(a, b)
} by {
    if r2_orthogonal(r2_neg(a), b) {
        r2_dot(r2_neg(a), b) = Real.0
        r2_dot_neg_left(a, b)
        r2_dot(r2_neg(a), b) = -r2_dot(a, b)
        -r2_dot(a, b) = Real.0
        r2_dot(a, b) = Real.0
        r2_orthogonal(a, b)
    }
}

/// Rearrangement of the six terms that occur in the two-coordinate expansion.
theorem real_pythagoras_rearrange(xx: Real, yy: Real, uu: Real, vv: Real, xu: Real, yv: Real) {
    (xx + xu + (xu + uu)) + (yy + yv + (yv + vv)) =
    (xx + yy) + (uu + vv) + (xu + yv) + (xu + yv)
} by {
    real_add_repeated_middle(xx, xu, uu)
    real_add_repeated_middle(yy, yv, vv)
    xx + xu + (xu + uu) = xx + (xu + xu) + uu
    yy + yv + (yv + vv) = yy + (yv + yv) + vv
    real_double_pair_rearrange(xu, yv)
    (xu + xu) + (yv + yv) = (xu + yv) + (xu + yv)
    real_add_six_pair_rearrange(xx, xu + xu, uu, yy, yv + yv, vv)
    (xx + (xu + xu) + uu) + (yy + (yv + yv) + vv) =
        (xx + yy) + (uu + vv) + ((xu + xu) + (yv + yv))
    (xx + yy) + (uu + vv) + ((xu + xu) + (yv + yv)) =
        (xx + yy) + (uu + vv) + (xu + yv) + (xu + yv)
}

/// Pythagoras for orthogonal vectors in the real coordinate plane.
theorem r2_pythagoras_vectors(a: Pair[Real, Real], b: Pair[Real, Real]) {
    r2_orthogonal(a, b) implies
    r2_norm_sq(r2_add(a, b)) = r2_norm_sq(a) + r2_norm_sq(b)
} by {
    if r2_orthogonal(a, b) {
        r2_dot(a, b) = Real.0
        a.first * b.first + a.second * b.second = Real.0

        r2_add(a, b).first = a.first + b.first
        r2_add(a, b).second = a.second + b.second
        r2_norm_sq(r2_add(a, b)) =
            (a.first + b.first) * (a.first + b.first) +
            (a.second + b.second) * (a.second + b.second)
        r2_norm_sq(a) = a.first * a.first + a.second * a.second
        r2_norm_sq(b) = b.first * b.first + b.second * b.second
        real_square_add(a.first, b.first)
        real_square_add(a.second, b.second)
        (a.first + b.first) * (a.first + b.first) =
            a.first * a.first + a.first * b.first + (a.first * b.first + b.first * b.first)
        (a.second + b.second) * (a.second + b.second) =
            a.second * a.second + a.second * b.second + (a.second * b.second + b.second * b.second)
        real_pythagoras_rearrange(
            a.first * a.first,
            a.second * a.second,
            b.first * b.first,
            b.second * b.second,
            a.first * b.first,
            a.second * b.second)
        (a.first + b.first) * (a.first + b.first) +
            (a.second + b.second) * (a.second + b.second) =
            (a.first * a.first + a.second * a.second) +
            (b.first * b.first + b.second * b.second) +
            (a.first * b.first + a.second * b.second) +
            (a.first * b.first + a.second * b.second)
        (a.first * b.first + a.second * b.second) +
            (a.first * b.first + a.second * b.second) = Real.0
        (a.first * a.first + a.second * a.second) +
            (b.first * b.first + b.second * b.second) + Real.0 =
            (a.first * a.first + a.second * a.second) +
            (b.first * b.first + b.second * b.second)
        r2_norm_sq(r2_add(a, b)) = r2_norm_sq(a) + r2_norm_sq(b)
    }
}

/// Subtracting through an intermediate point decomposes a displacement.
theorem r2_sub_through(a: Pair[Real, Real], b: Pair[Real, Real], c: Pair[Real, Real]) {
    r2_sub(c, a) = r2_add(r2_sub(b, a), r2_sub(c, b))
} by {
    let lhs = r2_sub(c, a)
    let rhs = r2_add(r2_sub(b, a), r2_sub(c, b))
    lhs.first = c.first - a.first
    lhs.second = c.second - a.second
    rhs.first = r2_sub(b, a).first + r2_sub(c, b).first
    r2_sub(b, a).first = b.first - a.first
    r2_sub(c, b).first = c.first - b.first
    rhs.first = (b.first - a.first) + (c.first - b.first)
    real_sub_through(a.first, b.first, c.first)
    (b.first - a.first) + (c.first - b.first) = c.first - a.first
    rhs.first = lhs.first
    rhs.second = r2_sub(b, a).second + r2_sub(c, b).second
    r2_sub(b, a).second = b.second - a.second
    r2_sub(c, b).second = c.second - b.second
    rhs.second = (b.second - a.second) + (c.second - b.second)
    real_sub_through(a.second, b.second, c.second)
    (b.second - a.second) + (c.second - b.second) = c.second - a.second
    rhs.second = lhs.second
    pair_ext(lhs, rhs)
}

/// Pythagoras for three points in the real coordinate plane, with the direction from `a` to `b`.
theorem r2_pythagoras_points(a: Pair[Real, Real], b: Pair[Real, Real], c: Pair[Real, Real]) {
    r2_orthogonal(r2_sub(b, a), r2_sub(c, b)) implies
    r2_norm_sq(r2_sub(c, a)) = r2_norm_sq(r2_sub(b, a)) + r2_norm_sq(r2_sub(c, b))
} by {
    if r2_orthogonal(r2_sub(b, a), r2_sub(c, b)) {
        r2_sub_through(a, b, c)
        r2_sub(c, a) = r2_add(r2_sub(b, a), r2_sub(c, b))
        r2_norm_sq(r2_sub(c, a)) = r2_norm_sq(r2_add(r2_sub(b, a), r2_sub(c, b)))
        r2_pythagoras_vectors(r2_sub(b, a), r2_sub(c, b))
        r2_norm_sq(r2_add(r2_sub(b, a), r2_sub(c, b))) =
            r2_norm_sq(r2_sub(b, a)) + r2_norm_sq(r2_sub(c, b))
    }
}

/// Freek Top 100 #4: the Pythagorean theorem in the real coordinate plane.
theorem top100_004_pythagorean_theorem(
    a: Pair[Real, Real], b: Pair[Real, Real], c: Pair[Real, Real]
) {
    r2_orthogonal(r2_sub(a, b), r2_sub(c, b)) implies
    r2_norm_sq(r2_sub(c, a)) = r2_norm_sq(r2_sub(b, a)) + r2_norm_sq(r2_sub(c, b))
} by {
    if r2_orthogonal(r2_sub(a, b), r2_sub(c, b)) {
        r2_sub_reverse_neg(a, b)
        r2_sub(a, b) = r2_neg(r2_sub(b, a))
        r2_orthogonal(r2_neg(r2_sub(b, a)), r2_sub(c, b))
        r2_orthogonal_neg_left(r2_sub(b, a), r2_sub(c, b))
        r2_orthogonal(r2_sub(b, a), r2_sub(c, b))
        r2_pythagoras_points(a, b, c)
    }
}
