from nat import Nat
from data.basic.functions import function_extensionality
from real import Real

numerals Real

/// The coefficient function obtained by dropping the constant coefficient.
define coeff_tail(c: Nat -> Real, i: Nat) -> Real {
    c(i.suc)
}

/// Evaluation of the polynomial determined by the first `n` coefficients.
define coeff_eval(c: Nat -> Real, x: Real, n: Nat) -> Real {
    match n {
        Nat.zero {
            Real.0
        }
        Nat.suc(pred) {
            c(Nat.0) + x * coeff_eval(coeff_tail(c), x, pred)
        }
    }
}

/// The synthetic quotient coefficients for division by `X - a`, using the first `n` coefficients.
define coeff_quotient(c: Nat -> Real, a: Real, n: Nat, i: Nat) -> Real {
    match n {
        Nat.zero {
            Real.0
        }
        Nat.suc(pred) {
            match i {
                Nat.zero {
                    coeff_eval(coeff_tail(c), a, pred)
                }
                Nat.suc(ipred) {
                    coeff_quotient(coeff_tail(c), a, pred, ipred)
                }
            }
        }
    }
}

/// The value at `x` of the product of a coefficient polynomial with `X - a`.
define coeff_mul_x_minus_a(q: Nat -> Real, a: Real, x: Real, n: Nat) -> Real {
    coeff_eval(q, x, n) * (x - a)
}

/// The constant coefficient of the synthetic quotient is the tail evaluated at the root.
theorem coeff_quotient_zero_index(c: Nat -> Real, a: Real, n: Nat) {
    coeff_quotient(c, a, n.suc, Nat.0) = coeff_eval(coeff_tail(c), a, n)
}

/// True when a coefficient polynomial is divisible by `X - a`.
define coeff_divisible_by_x_minus_a(c: Nat -> Real, a: Real, n: Nat) -> Bool {
    exists(q: Nat -> Real) {
        forall(x: Real) {
            coeff_eval(c, x, n) = coeff_mul_x_minus_a(q, a, x, n)
        }
    }
}

/// Adding the displacement from `a` to `x` to `a` gives `x`.
theorem real_add_sub_cancel_left(a: Real, x: Real) {
    a + (x - a) = x
} by {
    x - a = x + -a
    a + (x - a) = a + (x + -a)
    a + (x + -a) = a + (-a + x)
    a + (-a + x) = (a + -a) + x
    a + -a = Real.0
    (a + -a) + x = Real.0 + x
    Real.0 + x = x
}

/// Multiplication by `x` splits through an intermediate scalar `a`.
theorem real_mul_sub_split(a: Real, x: Real, t: Real) {
    x * t = a * t + (x - a) * t
} by {
    real_add_sub_cancel_left(a, x)
    x = a + (x - a)
    x * t = (a + (x - a)) * t
    (a + (x - a)) * t = a * t + (x - a) * t
}

/// The tail of the synthetic quotient is the synthetic quotient of the tail.
theorem coeff_tail_quotient(c: Nat -> Real, a: Real, n: Nat) {
    coeff_tail(coeff_quotient(c, a, n.suc), Nat.0) = coeff_quotient(coeff_tail(c), a, n, Nat.0) and
    coeff_tail(coeff_quotient(c, a, n.suc)) = coeff_quotient(coeff_tail(c), a, n)
} by {
    coeff_tail(coeff_quotient(c, a, n.suc), Nat.0) = coeff_quotient(c, a, n.suc, Nat.1)
    coeff_quotient(c, a, n.suc, Nat.1) = coeff_quotient(coeff_tail(c), a, n, Nat.0)
    forall(i: Nat) {
        coeff_tail(coeff_quotient(c, a, n.suc), i) = coeff_quotient(c, a, n.suc, i.suc)
        coeff_quotient(c, a, n.suc, i.suc) = coeff_quotient(coeff_tail(c), a, n, i)
        coeff_tail(coeff_quotient(c, a, n.suc), i) = coeff_quotient(coeff_tail(c), a, n, i)
    }
    function_extensionality(coeff_tail(coeff_quotient(c, a, n.suc)), coeff_quotient(coeff_tail(c), a, n))
}

/// The bounded remainder identity for a fixed coefficient function.
define coeff_remainder_at(c: Nat -> Real, a: Real, x: Real, n: Nat) -> Bool {
    coeff_eval(c, x, n) = coeff_eval(c, a, n) + (x - a) * coeff_eval(coeff_quotient(c, a, n), x, n)
}

/// The bounded remainder identity for all coefficient functions at a fixed bound.
define coeff_remainder_property(a: Real, x: Real, n: Nat) -> Bool {
    forall(c: Nat -> Real) {
        coeff_remainder_at(c, a, x, n)
    }
}

/// A universal bounded remainder property applies to any coefficient function.
theorem coeff_remainder_property_apply(a: Real, x: Real, n: Nat, c: Nat -> Real) {
    coeff_remainder_property(a, x, n) implies coeff_remainder_at(c, a, x, n)
} by {
    if coeff_remainder_property(a, x, n) {
        coeff_remainder_property(a, x, n) = forall(d: Nat -> Real) {
            coeff_remainder_at(d, a, x, n)
        }
        coeff_remainder_at(c, a, x, n)
    }
}

/// A pointwise proof gives the universal bounded remainder property.
theorem coeff_remainder_property_intro(a: Real, x: Real, n: Nat) {
    (forall(c: Nat -> Real) { coeff_remainder_at(c, a, x, n) }) implies
    coeff_remainder_property(a, x, n)
} by {
    forall(c: Nat -> Real) {
        coeff_remainder_at(c, a, x, n)
    }
    coeff_remainder_property(a, x, n) = forall(c: Nat -> Real) {
        coeff_remainder_at(c, a, x, n)
    }
}

/// The bounded coefficient-form remainder theorem, uniformly in the coefficients.
theorem coeff_remainder_property_all(a: Real, x: Real, n: Nat) {
    coeff_remainder_property(a, x, n)
} by {
    define statement(k: Nat) -> Bool {
        coeff_remainder_property(a, x, k)
    }

    forall(d: Nat -> Real) {
        coeff_eval(d, x, Nat.0) = Real.0
        coeff_eval(d, a, Nat.0) = Real.0
        coeff_eval(coeff_quotient(d, a, Nat.0), x, Nat.0) = Real.0
        Real.0 + (x - a) * Real.0 = Real.0
        coeff_eval(d, x, Nat.0) =
            coeff_eval(d, a, Nat.0) + (x - a) * coeff_eval(coeff_quotient(d, a, Nat.0), x, Nat.0)
        coeff_remainder_at(d, a, x, Nat.0)
    }
    coeff_remainder_property_intro(a, x, Nat.0)
    coeff_remainder_property(a, x, Nat.0)
    statement(Nat.0)

    forall(k: Nat) {
        if statement(k) {
            forall(d: Nat -> Real) {
                let dt = coeff_tail(d)
                let q = coeff_quotient(d, a, k.suc)
                let qt = coeff_quotient(dt, a, k)
                coeff_tail_quotient(d, a, k)
                coeff_tail(q) = qt
                q = coeff_quotient(d, a, k.suc)
                q(Nat.0) = coeff_quotient(d, a, k.suc, Nat.0)
                coeff_quotient_zero_index(d, a, k)
                coeff_quotient(d, a, k.suc, Nat.0) = coeff_eval(dt, a, k)
                q(Nat.0) = coeff_eval(dt, a, k)
                coeff_eval(q, x, k.suc) = q(Nat.0) + x * coeff_eval(coeff_tail(q), x, k)
                coeff_eval(q, x, k.suc) = coeff_eval(dt, a, k) + x * coeff_eval(qt, x, k)

                coeff_remainder_property_apply(a, x, k, dt)
                coeff_remainder_at(dt, a, x, k)
                coeff_eval(dt, x, k) = coeff_eval(dt, a, k) + (x - a) * coeff_eval(qt, x, k)

                coeff_eval(d, x, k.suc) = d(Nat.0) + x * coeff_eval(dt, x, k)
                coeff_eval(d, a, k.suc) = d(Nat.0) + a * coeff_eval(dt, a, k)
                coeff_eval(d, x, k.suc) = d(Nat.0) + x * (coeff_eval(dt, a, k) + (x - a) * coeff_eval(qt, x, k))
                d(Nat.0) + x * (coeff_eval(dt, a, k) + (x - a) * coeff_eval(qt, x, k)) =
                    d(Nat.0) + x * coeff_eval(dt, a, k) + x * ((x - a) * coeff_eval(qt, x, k))
                x * ((x - a) * coeff_eval(qt, x, k)) = (x - a) * (x * coeff_eval(qt, x, k))
                d(Nat.0) + x * coeff_eval(dt, a, k) + x * ((x - a) * coeff_eval(qt, x, k)) =
                    d(Nat.0) + x * coeff_eval(dt, a, k) + (x - a) * (x * coeff_eval(qt, x, k))
                real_mul_sub_split(a, x, coeff_eval(dt, a, k))
                x * coeff_eval(dt, a, k) = a * coeff_eval(dt, a, k) + (x - a) * coeff_eval(dt, a, k)
                d(Nat.0) + x * coeff_eval(dt, a, k) =
                    d(Nat.0) + a * coeff_eval(dt, a, k) + (x - a) * coeff_eval(dt, a, k)
                d(Nat.0) + x * coeff_eval(dt, a, k) + (x - a) * (x * coeff_eval(qt, x, k)) =
                    d(Nat.0) + a * coeff_eval(dt, a, k) + (x - a) * coeff_eval(dt, a, k) +
                    (x - a) * (x * coeff_eval(qt, x, k))
                (x - a) * coeff_eval(dt, a, k) + (x - a) * (x * coeff_eval(qt, x, k)) =
                    (x - a) * (coeff_eval(dt, a, k) + x * coeff_eval(qt, x, k))
                d(Nat.0) + a * coeff_eval(dt, a, k) + (x - a) * coeff_eval(dt, a, k) +
                    (x - a) * (x * coeff_eval(qt, x, k)) =
                    d(Nat.0) + a * coeff_eval(dt, a, k) +
                    (x - a) * (coeff_eval(dt, a, k) + x * coeff_eval(qt, x, k))
                coeff_eval(d, x, k.suc) = coeff_eval(d, a, k.suc) + (x - a) * coeff_eval(q, x, k.suc)
                coeff_remainder_at(d, a, x, k.suc)
            }
            coeff_remainder_property_intro(a, x, k.suc)
            coeff_remainder_property(a, x, k.suc)
            statement(k.suc)
        }
    }

    Nat.induction(statement)
    statement(n)
}

/// The coefficient-form remainder theorem.
theorem top100_089_remainder_theorem(c: Nat -> Real, a: Real, x: Real, n: Nat) {
    coeff_eval(c, x, n) = coeff_eval(c, a, n) + (x - a) * coeff_eval(coeff_quotient(c, a, n), x, n)
} by {
    coeff_remainder_property_all(a, x, n)
    coeff_remainder_property_apply(a, x, n, c)
    coeff_remainder_at(c, a, x, n)
}

/// A root gives divisibility by the corresponding linear factor.
theorem coeff_factor_theorem_forward(c: Nat -> Real, a: Real, n: Nat) {
    coeff_eval(c, a, n) = Real.0 implies coeff_divisible_by_x_minus_a(c, a, n)
} by {
    if coeff_eval(c, a, n) = Real.0 {
        forall(x: Real) {
            top100_089_remainder_theorem(c, a, x, n)
            coeff_eval(c, x, n) = coeff_eval(c, a, n) + (x - a) * coeff_eval(coeff_quotient(c, a, n), x, n)
            coeff_eval(c, a, n) = Real.0
            Real.0 + (x - a) * coeff_eval(coeff_quotient(c, a, n), x, n) =
                (x - a) * coeff_eval(coeff_quotient(c, a, n), x, n)
            (x - a) * coeff_eval(coeff_quotient(c, a, n), x, n) =
                coeff_eval(coeff_quotient(c, a, n), x, n) * (x - a)
            coeff_mul_x_minus_a(coeff_quotient(c, a, n), a, x, n) =
                coeff_eval(coeff_quotient(c, a, n), x, n) * (x - a)
            coeff_eval(c, x, n) = coeff_mul_x_minus_a(coeff_quotient(c, a, n), a, x, n)
        }
        exists(q: Nat -> Real) {
            forall(x: Real) {
                coeff_eval(c, x, n) = coeff_mul_x_minus_a(q, a, x, n)
            }
        }
        coeff_divisible_by_x_minus_a(c, a, n)
    }
}

/// Divisibility by `X - a` forces `a` to be a root.
theorem coeff_factor_theorem_reverse(c: Nat -> Real, a: Real, n: Nat) {
    coeff_divisible_by_x_minus_a(c, a, n) implies coeff_eval(c, a, n) = Real.0
} by {
    if coeff_divisible_by_x_minus_a(c, a, n) {
        let q: Nat -> Real satisfy {
            forall(x: Real) {
                coeff_eval(c, x, n) = coeff_mul_x_minus_a(q, a, x, n)
            }
        }
        coeff_eval(c, a, n) = coeff_mul_x_minus_a(q, a, a, n)
        coeff_mul_x_minus_a(q, a, a, n) = coeff_eval(q, a, n) * (a - a)
        a - a = a + -a
        a + -a = Real.0
        a - a = Real.0
        coeff_eval(q, a, n) * Real.0 = Real.0
        coeff_eval(c, a, n) = Real.0
    }
}

/// Freek Top 100 #89: the factor theorem for real coefficient polynomials.
theorem top100_089_factor_theorem(c: Nat -> Real, a: Real, n: Nat) {
    coeff_eval(c, a, n) = Real.0 = coeff_divisible_by_x_minus_a(c, a, n)
} by {
    if coeff_eval(c, a, n) = Real.0 {
        coeff_factor_theorem_forward(c, a, n)
        coeff_divisible_by_x_minus_a(c, a, n)
    }
    if coeff_divisible_by_x_minus_a(c, a, n) {
        coeff_factor_theorem_reverse(c, a, n)
        coeff_eval(c, a, n) = Real.0
    }
}
