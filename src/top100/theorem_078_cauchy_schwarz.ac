from nat import Nat
from real import Real
from real import finite_cauchy_schwarz, finite_dot, finite_square_sum

/// Freek Top 100 #78: the finite real Cauchy-Schwarz inequality.
theorem top100_078_cauchy_schwarz(a: Nat -> Real, b: Nat -> Real, n: Nat) {
    finite_dot(a, b, n) * finite_dot(a, b, n) <=
        finite_square_sum(a, n) * finite_square_sum(b, n)
} by {
    finite_cauchy_schwarz(a, b, n)
}
