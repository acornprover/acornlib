from real import Real
from geometry import Point2, heron_linear_product, point2_heron_triangle_squared

numerals Real

/// True when `length` is the Euclidean length of a real coordinate vector.
define point2_real_length_witness_57(p: Point2[Real], length: Real) -> Bool {
    Real.0 <= length and length * length = p.norm_sq
}

/// Freek Top 100 #57: Heron's formula in squared doubled-area form.
theorem top100_057_herons_formula_squared(
    a: Point2[Real], b: Point2[Real], c: Point2[Real],
    side_a: Real, side_b: Real, side_c: Real
) {
    point2_real_length_witness_57(c.sub(b), side_a) and
    point2_real_length_witness_57(c.sub(a), side_b) and
    point2_real_length_witness_57(b.sub(a), side_c)
    implies
    heron_linear_product(side_a, side_b, side_c) =
        (a.triangle_area2(b, c) + a.triangle_area2(b, c)) *
        (a.triangle_area2(b, c) + a.triangle_area2(b, c))
} by {
    if point2_real_length_witness_57(c.sub(b), side_a) and
        point2_real_length_witness_57(c.sub(a), side_b) and
        point2_real_length_witness_57(b.sub(a), side_c) {
        side_a * side_a = c.sub(b).norm_sq
        side_b * side_b = c.sub(a).norm_sq
        side_c * side_c = b.sub(a).norm_sq
        point2_heron_triangle_squared(a, b, c, side_a, side_b, side_c)
        heron_linear_product(side_a, side_b, side_c) =
            (a.triangle_area2(b, c) + a.triangle_area2(b, c)) *
            (a.triangle_area2(b, c) + a.triangle_area2(b, c))
    }
}
