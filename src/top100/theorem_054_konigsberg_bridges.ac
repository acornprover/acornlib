from nat import Nat
from nat import add_assoc, add_one_right, one_plus_one
from list import List

numerals Nat

/// True if a natural number is odd.
define odd_nat(n: Nat) -> Bool {
    match n {
        Nat.zero { false }
        Nat.suc(k) { not odd_nat(k) }
    }
}

/// Successor toggles oddness.
theorem odd_nat_suc(n: Nat) {
    odd_nat(n.suc) = not odd_nat(n)
}

/// One is odd.
theorem odd_nat_one {
    odd_nat(Nat.1)
}

/// Three is odd.
theorem odd_nat_three {
    odd_nat(Nat.3)
}

/// Five is odd.
theorem odd_nat_five {
    odd_nat(Nat.5)
}

/// Adding one to a five-term sum may be associated to the first summand.
theorem nat_one_add_sum5(a: Nat, b: Nat, c: Nat, d: Nat, e: Nat) {
    Nat.1 + (a + b + c + d + e) = Nat.1 + a + b + c + d + e
} by {
    add_assoc(a, b, c)
    add_assoc(a, b + c, d)
    add_assoc(a, b + c + d, e)
    add_assoc(b, c, d)
    add_assoc(b, c + d, e)
    a + b + c + d + e = a + (b + c + d + e)
    Nat.1 + (a + b + c + d + e) = Nat.1 + (a + (b + c + d + e))
    add_assoc(Nat.1, a, b + c + d + e)
    add_assoc(Nat.1 + a, b, c)
    add_assoc(Nat.1 + a, b + c, d)
    add_assoc(Nat.1 + a, b + c + d, e)
    Nat.1 + a + (b + c + d + e) = Nat.1 + (a + (b + c + d + e))
    Nat.1 + a + b + c + d + e = Nat.1 + a + (b + c + d + e)
}

/// True if exactly one of two propositions holds.
define exactly_one(left: Bool, right: Bool) -> Bool {
    (left and not right) or (not left and right)
}

/// The four land masses in the Konigsberg bridge graph.
inductive KonigsbergVertex {
    /// The land mass labelled `0` in the classical finite graph statement.
    v0
    /// The land mass labelled `1` in the classical finite graph statement.
    v1
    /// The land mass labelled `2` in the classical finite graph statement.
    v2
    /// The land mass labelled `3` in the classical finite graph statement.
    v3
}

/// True if a land mass is exactly one of two endpoints.
define exactly_one_endpoint(start: KonigsbergVertex, end: KonigsbergVertex, v: KonigsbergVertex) -> Bool {
    exactly_one(start = v, end = v)
}

/// True if a land mass is one of two endpoints.
define endpoint_has(start: KonigsbergVertex, end: KonigsbergVertex, v: KonigsbergVertex) -> Bool {
    start = v or end = v
}

/// The characteristic value of equality with a land mass.
define vertex_indicator(x: KonigsbergVertex, v: KonigsbergVertex) -> Nat {
    if x = v { Nat.1 } else { Nat.0 }
}

/// The seven bridges in the Konigsberg bridge graph.
inductive KonigsbergBridge {
    /// The bridge whose endpoints are `v0` and `v1`.
    e10
    /// The bridge whose endpoints are `v0` and `v2`.
    e20
    /// The bridge whose endpoints are `v0` and `v3`.
    e30
    /// The first bridge whose endpoints are `v1` and `v2`.
    e40
    /// The second bridge whose endpoints are `v1` and `v2`.
    e50
    /// The first bridge whose endpoints are `v2` and `v3`.
    e60
    /// The second bridge whose endpoints are `v2` and `v3`.
    e70
}

/// A bridge crossed in one of its two possible directions.
inductive DirectedKonigsbergBridge {
    /// Bridge `e10`, crossed from `v0` to `v1`.
    e10_01
    /// Bridge `e10`, crossed from `v1` to `v0`.
    e10_10
    /// Bridge `e20`, crossed from `v0` to `v2`.
    e20_02
    /// Bridge `e20`, crossed from `v2` to `v0`.
    e20_20
    /// Bridge `e30`, crossed from `v0` to `v3`.
    e30_03
    /// Bridge `e30`, crossed from `v3` to `v0`.
    e30_30
    /// Bridge `e40`, crossed from `v1` to `v2`.
    e40_12
    /// Bridge `e40`, crossed from `v2` to `v1`.
    e40_21
    /// Bridge `e50`, crossed from `v1` to `v2`.
    e50_12
    /// Bridge `e50`, crossed from `v2` to `v1`.
    e50_21
    /// Bridge `e60`, crossed from `v2` to `v3`.
    e60_23
    /// Bridge `e60`, crossed from `v3` to `v2`.
    e60_32
    /// Bridge `e70`, crossed from `v2` to `v3`.
    e70_23
    /// Bridge `e70`, crossed from `v3` to `v2`.
    e70_32
}

/// The underlying bridge of a directed crossing.
define directed_bridge(step: DirectedKonigsbergBridge) -> KonigsbergBridge {
    match step {
        DirectedKonigsbergBridge.e10_01 { KonigsbergBridge.e10 }
        DirectedKonigsbergBridge.e10_10 { KonigsbergBridge.e10 }
        DirectedKonigsbergBridge.e20_02 { KonigsbergBridge.e20 }
        DirectedKonigsbergBridge.e20_20 { KonigsbergBridge.e20 }
        DirectedKonigsbergBridge.e30_03 { KonigsbergBridge.e30 }
        DirectedKonigsbergBridge.e30_30 { KonigsbergBridge.e30 }
        DirectedKonigsbergBridge.e40_12 { KonigsbergBridge.e40 }
        DirectedKonigsbergBridge.e40_21 { KonigsbergBridge.e40 }
        DirectedKonigsbergBridge.e50_12 { KonigsbergBridge.e50 }
        DirectedKonigsbergBridge.e50_21 { KonigsbergBridge.e50 }
        DirectedKonigsbergBridge.e60_23 { KonigsbergBridge.e60 }
        DirectedKonigsbergBridge.e60_32 { KonigsbergBridge.e60 }
        DirectedKonigsbergBridge.e70_23 { KonigsbergBridge.e70 }
        DirectedKonigsbergBridge.e70_32 { KonigsbergBridge.e70 }
    }
}

/// The source land mass of a directed crossing.
define directed_source(step: DirectedKonigsbergBridge) -> KonigsbergVertex {
    match step {
        DirectedKonigsbergBridge.e10_01 { KonigsbergVertex.v0 }
        DirectedKonigsbergBridge.e10_10 { KonigsbergVertex.v1 }
        DirectedKonigsbergBridge.e20_02 { KonigsbergVertex.v0 }
        DirectedKonigsbergBridge.e20_20 { KonigsbergVertex.v2 }
        DirectedKonigsbergBridge.e30_03 { KonigsbergVertex.v0 }
        DirectedKonigsbergBridge.e30_30 { KonigsbergVertex.v3 }
        DirectedKonigsbergBridge.e40_12 { KonigsbergVertex.v1 }
        DirectedKonigsbergBridge.e40_21 { KonigsbergVertex.v2 }
        DirectedKonigsbergBridge.e50_12 { KonigsbergVertex.v1 }
        DirectedKonigsbergBridge.e50_21 { KonigsbergVertex.v2 }
        DirectedKonigsbergBridge.e60_23 { KonigsbergVertex.v2 }
        DirectedKonigsbergBridge.e60_32 { KonigsbergVertex.v3 }
        DirectedKonigsbergBridge.e70_23 { KonigsbergVertex.v2 }
        DirectedKonigsbergBridge.e70_32 { KonigsbergVertex.v3 }
    }
}

/// The target land mass of a directed crossing.
define directed_target(step: DirectedKonigsbergBridge) -> KonigsbergVertex {
    match step {
        DirectedKonigsbergBridge.e10_01 { KonigsbergVertex.v1 }
        DirectedKonigsbergBridge.e10_10 { KonigsbergVertex.v0 }
        DirectedKonigsbergBridge.e20_02 { KonigsbergVertex.v2 }
        DirectedKonigsbergBridge.e20_20 { KonigsbergVertex.v0 }
        DirectedKonigsbergBridge.e30_03 { KonigsbergVertex.v3 }
        DirectedKonigsbergBridge.e30_30 { KonigsbergVertex.v0 }
        DirectedKonigsbergBridge.e40_12 { KonigsbergVertex.v2 }
        DirectedKonigsbergBridge.e40_21 { KonigsbergVertex.v1 }
        DirectedKonigsbergBridge.e50_12 { KonigsbergVertex.v2 }
        DirectedKonigsbergBridge.e50_21 { KonigsbergVertex.v1 }
        DirectedKonigsbergBridge.e60_23 { KonigsbergVertex.v3 }
        DirectedKonigsbergBridge.e60_32 { KonigsbergVertex.v2 }
        DirectedKonigsbergBridge.e70_23 { KonigsbergVertex.v3 }
        DirectedKonigsbergBridge.e70_32 { KonigsbergVertex.v2 }
    }
}

/// The number of directed crossings leaving a land mass.
define source_count(steps: List[DirectedKonigsbergBridge], v: KonigsbergVertex) -> Nat {
    match steps {
        List.nil { Nat.0 }
        List.cons(step, tail) {
            vertex_indicator(directed_source(step), v) + source_count(tail, v)
        }
    }
}

/// The number of directed crossings entering a land mass.
define target_count(steps: List[DirectedKonigsbergBridge], v: KonigsbergVertex) -> Nat {
    match steps {
        List.nil { Nat.0 }
        List.cons(step, tail) {
            vertex_indicator(directed_target(step), v) + target_count(tail, v)
        }
    }
}

/// The number of directed crossings incident to a land mass.
define directed_incidence_count(steps: List[DirectedKonigsbergBridge], v: KonigsbergVertex) -> Nat {
    source_count(steps, v) + target_count(steps, v)
}

/// True if a directed list of bridge crossings forms a walk from `start` to `end`.
define directed_konigsberg_walk(
    start: KonigsbergVertex, steps: List[DirectedKonigsbergBridge], end: KonigsbergVertex
) -> Bool {
    match steps {
        List.nil {
            start = end
        }
        List.cons(step, tail) {
            directed_source(step) = start and directed_konigsberg_walk(directed_target(step), tail, end)
        }
    }
}

/// The number of times an underlying bridge is used by a directed crossing list.
define directed_bridge_count(steps: List[DirectedKonigsbergBridge], bridge: KonigsbergBridge) -> Nat {
    match steps {
        List.nil { Nat.0 }
        List.cons(step, tail) {
            if directed_bridge(step) = bridge {
                Nat.1 + directed_bridge_count(tail, bridge)
            } else {
                directed_bridge_count(tail, bridge)
            }
        }
    }
}

/// True if every underlying Konigsberg bridge is used exactly once.
define directed_uses_every_bridge_once(steps: List[DirectedKonigsbergBridge]) -> Bool {
    directed_bridge_count(steps, KonigsbergBridge.e10) = Nat.1 and
    directed_bridge_count(steps, KonigsbergBridge.e20) = Nat.1 and
    directed_bridge_count(steps, KonigsbergBridge.e30) = Nat.1 and
    directed_bridge_count(steps, KonigsbergBridge.e40) = Nat.1 and
    directed_bridge_count(steps, KonigsbergBridge.e50) = Nat.1 and
    directed_bridge_count(steps, KonigsbergBridge.e60) = Nat.1 and
    directed_bridge_count(steps, KonigsbergBridge.e70) = Nat.1
}

/// The fixed odd degrees of the four Konigsberg land masses.
define konigsberg_directed_degree_counts(steps: List[DirectedKonigsbergBridge]) -> Bool {
    directed_incidence_count(steps, KonigsbergVertex.v0) = Nat.3 and
    directed_incidence_count(steps, KonigsbergVertex.v1) = Nat.3 and
    directed_incidence_count(steps, KonigsbergVertex.v2) = Nat.5 and
    directed_incidence_count(steps, KonigsbergVertex.v3) = Nat.3
}

/// True if a directed list of bridge crossings is an Eulerian trail in the Konigsberg graph.
define directed_konigsberg_eulerian_trail(steps: List[DirectedKonigsbergBridge]) -> Bool {
    directed_uses_every_bridge_once(steps) and
        exists(start: KonigsbergVertex, end: KonigsbergVertex) {
            directed_konigsberg_walk(start, steps, end)
        }
}

/// Incident counts split over the head of a directed crossing list.
theorem directed_incidence_count_cons(
    step: DirectedKonigsbergBridge, tail: List[DirectedKonigsbergBridge], v: KonigsbergVertex
) {
    directed_incidence_count(List.cons(step, tail), v) =
        vertex_indicator(directed_source(step), v) + vertex_indicator(directed_target(step), v) +
        directed_incidence_count(tail, v)
} by {
    directed_incidence_count(List.cons(step, tail), v) =
        source_count(List.cons(step, tail), v) + target_count(List.cons(step, tail), v)
    source_count(List.cons(step, tail), v) = vertex_indicator(directed_source(step), v) + source_count(tail, v)
    target_count(List.cons(step, tail), v) = vertex_indicator(directed_target(step), v) + target_count(tail, v)
    directed_incidence_count(tail, v) = source_count(tail, v) + target_count(tail, v)
    vertex_indicator(directed_source(step), v) + source_count(tail, v) +
        (vertex_indicator(directed_target(step), v) + target_count(tail, v)) =
        vertex_indicator(directed_source(step), v) + vertex_indicator(directed_target(step), v) +
        (source_count(tail, v) + target_count(tail, v))
}

/// A crossing of bridge `e10` from `v0` to `v1` adds one incident crossing at `v0`.
theorem directed_incidence_count_v0_cons_e10_01(tail: List[DirectedKonigsbergBridge]) {
    directed_incidence_count(List.cons(DirectedKonigsbergBridge.e10_01, tail), KonigsbergVertex.v0) =
        Nat.1 + directed_incidence_count(tail, KonigsbergVertex.v0)
} by {
    directed_incidence_count_cons(DirectedKonigsbergBridge.e10_01, tail, KonigsbergVertex.v0)
    vertex_indicator(directed_source(DirectedKonigsbergBridge.e10_01), KonigsbergVertex.v0) = Nat.1
    vertex_indicator(directed_target(DirectedKonigsbergBridge.e10_01), KonigsbergVertex.v0) = Nat.0
}

/// A crossing of bridge `e10` from `v1` to `v0` adds one incident crossing at `v0`.
theorem directed_incidence_count_v0_cons_e10_10(tail: List[DirectedKonigsbergBridge]) {
    directed_incidence_count(List.cons(DirectedKonigsbergBridge.e10_10, tail), KonigsbergVertex.v0) =
        Nat.1 + directed_incidence_count(tail, KonigsbergVertex.v0)
} by {
    directed_incidence_count_cons(DirectedKonigsbergBridge.e10_10, tail, KonigsbergVertex.v0)
    vertex_indicator(directed_source(DirectedKonigsbergBridge.e10_10), KonigsbergVertex.v0) = Nat.0
    vertex_indicator(directed_target(DirectedKonigsbergBridge.e10_10), KonigsbergVertex.v0) = Nat.1
}

/// A crossing of bridge `e20` from `v0` to `v2` adds one incident crossing at `v0`.
theorem directed_incidence_count_v0_cons_e20_02(tail: List[DirectedKonigsbergBridge]) {
    directed_incidence_count(List.cons(DirectedKonigsbergBridge.e20_02, tail), KonigsbergVertex.v0) =
        Nat.1 + directed_incidence_count(tail, KonigsbergVertex.v0)
} by {
    directed_incidence_count_cons(DirectedKonigsbergBridge.e20_02, tail, KonigsbergVertex.v0)
    vertex_indicator(directed_source(DirectedKonigsbergBridge.e20_02), KonigsbergVertex.v0) = Nat.1
    vertex_indicator(directed_target(DirectedKonigsbergBridge.e20_02), KonigsbergVertex.v0) = Nat.0
}

/// A crossing of bridge `e20` from `v2` to `v0` adds one incident crossing at `v0`.
theorem directed_incidence_count_v0_cons_e20_20(tail: List[DirectedKonigsbergBridge]) {
    directed_incidence_count(List.cons(DirectedKonigsbergBridge.e20_20, tail), KonigsbergVertex.v0) =
        Nat.1 + directed_incidence_count(tail, KonigsbergVertex.v0)
} by {
    directed_incidence_count_cons(DirectedKonigsbergBridge.e20_20, tail, KonigsbergVertex.v0)
    vertex_indicator(directed_source(DirectedKonigsbergBridge.e20_20), KonigsbergVertex.v0) = Nat.0
    vertex_indicator(directed_target(DirectedKonigsbergBridge.e20_20), KonigsbergVertex.v0) = Nat.1
}

/// A crossing of bridge `e30` from `v0` to `v3` adds one incident crossing at `v0`.
theorem directed_incidence_count_v0_cons_e30_03(tail: List[DirectedKonigsbergBridge]) {
    directed_incidence_count(List.cons(DirectedKonigsbergBridge.e30_03, tail), KonigsbergVertex.v0) =
        Nat.1 + directed_incidence_count(tail, KonigsbergVertex.v0)
} by {
    directed_incidence_count_cons(DirectedKonigsbergBridge.e30_03, tail, KonigsbergVertex.v0)
    vertex_indicator(directed_source(DirectedKonigsbergBridge.e30_03), KonigsbergVertex.v0) = Nat.1
    vertex_indicator(directed_target(DirectedKonigsbergBridge.e30_03), KonigsbergVertex.v0) = Nat.0
}

/// A crossing of bridge `e30` from `v3` to `v0` adds one incident crossing at `v0`.
theorem directed_incidence_count_v0_cons_e30_30(tail: List[DirectedKonigsbergBridge]) {
    directed_incidence_count(List.cons(DirectedKonigsbergBridge.e30_30, tail), KonigsbergVertex.v0) =
        Nat.1 + directed_incidence_count(tail, KonigsbergVertex.v0)
} by {
    directed_incidence_count_cons(DirectedKonigsbergBridge.e30_30, tail, KonigsbergVertex.v0)
    vertex_indicator(directed_source(DirectedKonigsbergBridge.e30_30), KonigsbergVertex.v0) = Nat.0
    vertex_indicator(directed_target(DirectedKonigsbergBridge.e30_30), KonigsbergVertex.v0) = Nat.1
}

/// A crossing of bridge `e40` does not touch `v0`.
theorem directed_incidence_count_v0_cons_e40_12(tail: List[DirectedKonigsbergBridge]) {
    directed_incidence_count(List.cons(DirectedKonigsbergBridge.e40_12, tail), KonigsbergVertex.v0) =
        directed_incidence_count(tail, KonigsbergVertex.v0)
} by {
    directed_incidence_count_cons(DirectedKonigsbergBridge.e40_12, tail, KonigsbergVertex.v0)
    vertex_indicator(directed_source(DirectedKonigsbergBridge.e40_12), KonigsbergVertex.v0) = Nat.0
    vertex_indicator(directed_target(DirectedKonigsbergBridge.e40_12), KonigsbergVertex.v0) = Nat.0
}

/// A crossing of bridge `e40` does not touch `v0`.
theorem directed_incidence_count_v0_cons_e40_21(tail: List[DirectedKonigsbergBridge]) {
    directed_incidence_count(List.cons(DirectedKonigsbergBridge.e40_21, tail), KonigsbergVertex.v0) =
        directed_incidence_count(tail, KonigsbergVertex.v0)
} by {
    directed_incidence_count_cons(DirectedKonigsbergBridge.e40_21, tail, KonigsbergVertex.v0)
    vertex_indicator(directed_source(DirectedKonigsbergBridge.e40_21), KonigsbergVertex.v0) = Nat.0
    vertex_indicator(directed_target(DirectedKonigsbergBridge.e40_21), KonigsbergVertex.v0) = Nat.0
}

/// A crossing of bridge `e50` does not touch `v0`.
theorem directed_incidence_count_v0_cons_e50_12(tail: List[DirectedKonigsbergBridge]) {
    directed_incidence_count(List.cons(DirectedKonigsbergBridge.e50_12, tail), KonigsbergVertex.v0) =
        directed_incidence_count(tail, KonigsbergVertex.v0)
} by {
    directed_incidence_count_cons(DirectedKonigsbergBridge.e50_12, tail, KonigsbergVertex.v0)
    vertex_indicator(directed_source(DirectedKonigsbergBridge.e50_12), KonigsbergVertex.v0) = Nat.0
    vertex_indicator(directed_target(DirectedKonigsbergBridge.e50_12), KonigsbergVertex.v0) = Nat.0
}

/// A crossing of bridge `e50` does not touch `v0`.
theorem directed_incidence_count_v0_cons_e50_21(tail: List[DirectedKonigsbergBridge]) {
    directed_incidence_count(List.cons(DirectedKonigsbergBridge.e50_21, tail), KonigsbergVertex.v0) =
        directed_incidence_count(tail, KonigsbergVertex.v0)
} by {
    directed_incidence_count_cons(DirectedKonigsbergBridge.e50_21, tail, KonigsbergVertex.v0)
    vertex_indicator(directed_source(DirectedKonigsbergBridge.e50_21), KonigsbergVertex.v0) = Nat.0
    vertex_indicator(directed_target(DirectedKonigsbergBridge.e50_21), KonigsbergVertex.v0) = Nat.0
}

/// A crossing of bridge `e60` does not touch `v0`.
theorem directed_incidence_count_v0_cons_e60_23(tail: List[DirectedKonigsbergBridge]) {
    directed_incidence_count(List.cons(DirectedKonigsbergBridge.e60_23, tail), KonigsbergVertex.v0) =
        directed_incidence_count(tail, KonigsbergVertex.v0)
} by {
    directed_incidence_count_cons(DirectedKonigsbergBridge.e60_23, tail, KonigsbergVertex.v0)
    vertex_indicator(directed_source(DirectedKonigsbergBridge.e60_23), KonigsbergVertex.v0) = Nat.0
    vertex_indicator(directed_target(DirectedKonigsbergBridge.e60_23), KonigsbergVertex.v0) = Nat.0
}

/// A crossing of bridge `e60` does not touch `v0`.
theorem directed_incidence_count_v0_cons_e60_32(tail: List[DirectedKonigsbergBridge]) {
    directed_incidence_count(List.cons(DirectedKonigsbergBridge.e60_32, tail), KonigsbergVertex.v0) =
        directed_incidence_count(tail, KonigsbergVertex.v0)
} by {
    directed_incidence_count_cons(DirectedKonigsbergBridge.e60_32, tail, KonigsbergVertex.v0)
    vertex_indicator(directed_source(DirectedKonigsbergBridge.e60_32), KonigsbergVertex.v0) = Nat.0
    vertex_indicator(directed_target(DirectedKonigsbergBridge.e60_32), KonigsbergVertex.v0) = Nat.0
}

/// A crossing of bridge `e70` does not touch `v0`.
theorem directed_incidence_count_v0_cons_e70_23(tail: List[DirectedKonigsbergBridge]) {
    directed_incidence_count(List.cons(DirectedKonigsbergBridge.e70_23, tail), KonigsbergVertex.v0) =
        directed_incidence_count(tail, KonigsbergVertex.v0)
} by {
    directed_incidence_count_cons(DirectedKonigsbergBridge.e70_23, tail, KonigsbergVertex.v0)
    vertex_indicator(directed_source(DirectedKonigsbergBridge.e70_23), KonigsbergVertex.v0) = Nat.0
    vertex_indicator(directed_target(DirectedKonigsbergBridge.e70_23), KonigsbergVertex.v0) = Nat.0
}

/// A crossing of bridge `e70` does not touch `v0`.
theorem directed_incidence_count_v0_cons_e70_32(tail: List[DirectedKonigsbergBridge]) {
    directed_incidence_count(List.cons(DirectedKonigsbergBridge.e70_32, tail), KonigsbergVertex.v0) =
        directed_incidence_count(tail, KonigsbergVertex.v0)
} by {
    directed_incidence_count_cons(DirectedKonigsbergBridge.e70_32, tail, KonigsbergVertex.v0)
    vertex_indicator(directed_source(DirectedKonigsbergBridge.e70_32), KonigsbergVertex.v0) = Nat.0
    vertex_indicator(directed_target(DirectedKonigsbergBridge.e70_32), KonigsbergVertex.v0) = Nat.0
}

/// The incident crossings at `v0` are exactly the uses of bridges `e10`, `e20`, and `e30`.
theorem directed_incidence_count_v0(steps: List[DirectedKonigsbergBridge]) {
    directed_incidence_count(steps, KonigsbergVertex.v0) =
        directed_bridge_count(steps, KonigsbergBridge.e10) +
        directed_bridge_count(steps, KonigsbergBridge.e20) +
        directed_bridge_count(steps, KonigsbergBridge.e30)
} by {
    define p(xs: List[DirectedKonigsbergBridge]) -> Bool {
        directed_incidence_count(xs, KonigsbergVertex.v0) =
            directed_bridge_count(xs, KonigsbergBridge.e10) +
            directed_bridge_count(xs, KonigsbergBridge.e20) +
            directed_bridge_count(xs, KonigsbergBridge.e30)
    }

    directed_incidence_count(List.nil[DirectedKonigsbergBridge], KonigsbergVertex.v0) = Nat.0
    directed_bridge_count(List.nil[DirectedKonigsbergBridge], KonigsbergBridge.e10) = Nat.0
    directed_bridge_count(List.nil[DirectedKonigsbergBridge], KonigsbergBridge.e20) = Nat.0
    directed_incidence_count(List.nil[DirectedKonigsbergBridge], KonigsbergVertex.v3) = Nat.0
    directed_bridge_count(List.nil[DirectedKonigsbergBridge], KonigsbergBridge.e30) = Nat.0
    directed_bridge_count(List.nil[DirectedKonigsbergBridge], KonigsbergBridge.e60) = Nat.0
    directed_bridge_count(List.nil[DirectedKonigsbergBridge], KonigsbergBridge.e70) = Nat.0
    Nat.0 = Nat.0 + Nat.0 + Nat.0
    p(List.nil[DirectedKonigsbergBridge])

    forall(step: DirectedKonigsbergBridge, tail: List[DirectedKonigsbergBridge]) {
        if p(tail) {
            match step {
                DirectedKonigsbergBridge.e10_01 {
                    directed_incidence_count_v0_cons_e10_01(tail)
                    let a = directed_bridge_count(tail, KonigsbergBridge.e10)
                    let b = directed_bridge_count(tail, KonigsbergBridge.e20)
                    let c = directed_bridge_count(tail, KonigsbergBridge.e30)
                    directed_incidence_count(tail, KonigsbergVertex.v0) = a + b + c
                    Nat.1 + directed_incidence_count(tail, KonigsbergVertex.v0) = Nat.1 + (a + b + c)
                    Nat.1 + (a + b + c) = Nat.1 + a + b + c
                    directed_bridge_count(List.cons(DirectedKonigsbergBridge.e10_01, tail), KonigsbergBridge.e10) = Nat.1 + directed_bridge_count(tail, KonigsbergBridge.e10)
                    directed_bridge_count(List.cons(DirectedKonigsbergBridge.e10_01, tail), KonigsbergBridge.e20) = directed_bridge_count(tail, KonigsbergBridge.e20)
                    directed_bridge_count(List.cons(DirectedKonigsbergBridge.e10_01, tail), KonigsbergBridge.e30) = directed_bridge_count(tail, KonigsbergBridge.e30)
                    p(List.cons(DirectedKonigsbergBridge.e10_01, tail))
                    p(List.cons(step, tail))
                }
                DirectedKonigsbergBridge.e10_10 {
                    directed_incidence_count_v0_cons_e10_10(tail)
                    let a = directed_bridge_count(tail, KonigsbergBridge.e10)
                    let b = directed_bridge_count(tail, KonigsbergBridge.e20)
                    let c = directed_bridge_count(tail, KonigsbergBridge.e30)
                    directed_incidence_count(tail, KonigsbergVertex.v0) = a + b + c
                    Nat.1 + directed_incidence_count(tail, KonigsbergVertex.v0) = Nat.1 + (a + b + c)
                    Nat.1 + (a + b + c) = Nat.1 + a + b + c
                    directed_bridge_count(List.cons(DirectedKonigsbergBridge.e10_10, tail), KonigsbergBridge.e10) = Nat.1 + directed_bridge_count(tail, KonigsbergBridge.e10)
                    directed_bridge_count(List.cons(DirectedKonigsbergBridge.e10_10, tail), KonigsbergBridge.e20) = directed_bridge_count(tail, KonigsbergBridge.e20)
                    directed_bridge_count(List.cons(DirectedKonigsbergBridge.e10_10, tail), KonigsbergBridge.e30) = directed_bridge_count(tail, KonigsbergBridge.e30)
                    p(List.cons(DirectedKonigsbergBridge.e10_10, tail))
                    p(List.cons(step, tail))
                }
                DirectedKonigsbergBridge.e20_02 {
                    directed_incidence_count_v0_cons_e20_02(tail)
                    let a = directed_bridge_count(tail, KonigsbergBridge.e10)
                    let b = directed_bridge_count(tail, KonigsbergBridge.e20)
                    let c = directed_bridge_count(tail, KonigsbergBridge.e30)
                    directed_incidence_count(tail, KonigsbergVertex.v0) = a + b + c
                    Nat.1 + directed_incidence_count(tail, KonigsbergVertex.v0) = Nat.1 + (a + b + c)
                    Nat.1 + (a + b + c) = Nat.1 + a + b + c
                    Nat.1 + a = a + Nat.1
                    Nat.1 + a + b + c = a + Nat.1 + b + c
                    a + Nat.1 + b + c = a + (Nat.1 + b) + c
                    directed_bridge_count(List.cons(DirectedKonigsbergBridge.e20_02, tail), KonigsbergBridge.e10) = directed_bridge_count(tail, KonigsbergBridge.e10)
                    directed_bridge_count(List.cons(DirectedKonigsbergBridge.e20_02, tail), KonigsbergBridge.e20) = Nat.1 + directed_bridge_count(tail, KonigsbergBridge.e20)
                    directed_bridge_count(List.cons(DirectedKonigsbergBridge.e20_02, tail), KonigsbergBridge.e30) = directed_bridge_count(tail, KonigsbergBridge.e30)
                    p(List.cons(DirectedKonigsbergBridge.e20_02, tail))
                    p(List.cons(step, tail))
                }
                DirectedKonigsbergBridge.e20_20 {
                    directed_incidence_count_v0_cons_e20_20(tail)
                    let a = directed_bridge_count(tail, KonigsbergBridge.e10)
                    let b = directed_bridge_count(tail, KonigsbergBridge.e20)
                    let c = directed_bridge_count(tail, KonigsbergBridge.e30)
                    directed_incidence_count(tail, KonigsbergVertex.v0) = a + b + c
                    Nat.1 + directed_incidence_count(tail, KonigsbergVertex.v0) = Nat.1 + (a + b + c)
                    Nat.1 + (a + b + c) = Nat.1 + a + b + c
                    Nat.1 + a = a + Nat.1
                    Nat.1 + a + b + c = a + Nat.1 + b + c
                    a + Nat.1 + b + c = a + (Nat.1 + b) + c
                    directed_bridge_count(List.cons(DirectedKonigsbergBridge.e20_20, tail), KonigsbergBridge.e10) = directed_bridge_count(tail, KonigsbergBridge.e10)
                    directed_bridge_count(List.cons(DirectedKonigsbergBridge.e20_20, tail), KonigsbergBridge.e20) = Nat.1 + directed_bridge_count(tail, KonigsbergBridge.e20)
                    directed_bridge_count(List.cons(DirectedKonigsbergBridge.e20_20, tail), KonigsbergBridge.e30) = directed_bridge_count(tail, KonigsbergBridge.e30)
                    p(List.cons(DirectedKonigsbergBridge.e20_20, tail))
                    p(List.cons(step, tail))
                }
                DirectedKonigsbergBridge.e30_03 {
                    directed_incidence_count_v0_cons_e30_03(tail)
                    let a = directed_bridge_count(tail, KonigsbergBridge.e10)
                    let b = directed_bridge_count(tail, KonigsbergBridge.e20)
                    let c = directed_bridge_count(tail, KonigsbergBridge.e30)
                    directed_incidence_count(tail, KonigsbergVertex.v0) = a + b + c
                    Nat.1 + directed_incidence_count(tail, KonigsbergVertex.v0) = Nat.1 + (a + b + c)
                    Nat.1 + (a + b + c) = Nat.1 + a + b + c
                    Nat.1 + a = a + Nat.1
                    Nat.1 + a + b + c = a + Nat.1 + b + c
                    Nat.1 + b = b + Nat.1
                    a + Nat.1 + b + c = a + b + Nat.1 + c
                    a + b + Nat.1 + c = a + b + (Nat.1 + c)
                    directed_bridge_count(List.cons(DirectedKonigsbergBridge.e30_03, tail), KonigsbergBridge.e10) = directed_bridge_count(tail, KonigsbergBridge.e10)
                    directed_bridge_count(List.cons(DirectedKonigsbergBridge.e30_03, tail), KonigsbergBridge.e20) = directed_bridge_count(tail, KonigsbergBridge.e20)
                    directed_bridge_count(List.cons(DirectedKonigsbergBridge.e30_03, tail), KonigsbergBridge.e30) = Nat.1 + directed_bridge_count(tail, KonigsbergBridge.e30)
                    p(List.cons(DirectedKonigsbergBridge.e30_03, tail))
                    p(List.cons(step, tail))
                }
                DirectedKonigsbergBridge.e30_30 {
                    directed_incidence_count_v0_cons_e30_30(tail)
                    let a = directed_bridge_count(tail, KonigsbergBridge.e10)
                    let b = directed_bridge_count(tail, KonigsbergBridge.e20)
                    let c = directed_bridge_count(tail, KonigsbergBridge.e30)
                    directed_incidence_count(tail, KonigsbergVertex.v0) = a + b + c
                    Nat.1 + directed_incidence_count(tail, KonigsbergVertex.v0) = Nat.1 + (a + b + c)
                    Nat.1 + (a + b + c) = Nat.1 + a + b + c
                    Nat.1 + a = a + Nat.1
                    Nat.1 + a + b + c = a + Nat.1 + b + c
                    Nat.1 + b = b + Nat.1
                    a + Nat.1 + b + c = a + b + Nat.1 + c
                    a + b + Nat.1 + c = a + b + (Nat.1 + c)
                    directed_bridge_count(List.cons(DirectedKonigsbergBridge.e30_30, tail), KonigsbergBridge.e10) = directed_bridge_count(tail, KonigsbergBridge.e10)
                    directed_bridge_count(List.cons(DirectedKonigsbergBridge.e30_30, tail), KonigsbergBridge.e20) = directed_bridge_count(tail, KonigsbergBridge.e20)
                    directed_bridge_count(List.cons(DirectedKonigsbergBridge.e30_30, tail), KonigsbergBridge.e30) = Nat.1 + directed_bridge_count(tail, KonigsbergBridge.e30)
                    p(List.cons(DirectedKonigsbergBridge.e30_30, tail))
                    p(List.cons(step, tail))
                }
                DirectedKonigsbergBridge.e40_12 {
                    directed_incidence_count_v0_cons_e40_12(tail)
                    directed_bridge_count(List.cons(DirectedKonigsbergBridge.e40_12, tail), KonigsbergBridge.e10) = directed_bridge_count(tail, KonigsbergBridge.e10)
                    directed_bridge_count(List.cons(DirectedKonigsbergBridge.e40_12, tail), KonigsbergBridge.e20) = directed_bridge_count(tail, KonigsbergBridge.e20)
                    directed_bridge_count(List.cons(DirectedKonigsbergBridge.e40_12, tail), KonigsbergBridge.e30) = directed_bridge_count(tail, KonigsbergBridge.e30)
                    p(List.cons(DirectedKonigsbergBridge.e40_12, tail))
                    p(List.cons(step, tail))
                }
                DirectedKonigsbergBridge.e40_21 {
                    directed_incidence_count_v0_cons_e40_21(tail)
                    directed_bridge_count(List.cons(DirectedKonigsbergBridge.e40_21, tail), KonigsbergBridge.e10) = directed_bridge_count(tail, KonigsbergBridge.e10)
                    directed_bridge_count(List.cons(DirectedKonigsbergBridge.e40_21, tail), KonigsbergBridge.e20) = directed_bridge_count(tail, KonigsbergBridge.e20)
                    directed_bridge_count(List.cons(DirectedKonigsbergBridge.e40_21, tail), KonigsbergBridge.e30) = directed_bridge_count(tail, KonigsbergBridge.e30)
                    p(List.cons(DirectedKonigsbergBridge.e40_21, tail))
                    p(List.cons(step, tail))
                }
                DirectedKonigsbergBridge.e50_12 {
                    directed_incidence_count_v0_cons_e50_12(tail)
                    directed_bridge_count(List.cons(DirectedKonigsbergBridge.e50_12, tail), KonigsbergBridge.e10) = directed_bridge_count(tail, KonigsbergBridge.e10)
                    directed_bridge_count(List.cons(DirectedKonigsbergBridge.e50_12, tail), KonigsbergBridge.e20) = directed_bridge_count(tail, KonigsbergBridge.e20)
                    directed_bridge_count(List.cons(DirectedKonigsbergBridge.e50_12, tail), KonigsbergBridge.e30) = directed_bridge_count(tail, KonigsbergBridge.e30)
                    p(List.cons(DirectedKonigsbergBridge.e50_12, tail))
                    p(List.cons(step, tail))
                }
                DirectedKonigsbergBridge.e50_21 {
                    directed_incidence_count_v0_cons_e50_21(tail)
                    directed_bridge_count(List.cons(DirectedKonigsbergBridge.e50_21, tail), KonigsbergBridge.e10) = directed_bridge_count(tail, KonigsbergBridge.e10)
                    directed_bridge_count(List.cons(DirectedKonigsbergBridge.e50_21, tail), KonigsbergBridge.e20) = directed_bridge_count(tail, KonigsbergBridge.e20)
                    directed_bridge_count(List.cons(DirectedKonigsbergBridge.e50_21, tail), KonigsbergBridge.e30) = directed_bridge_count(tail, KonigsbergBridge.e30)
                    p(List.cons(DirectedKonigsbergBridge.e50_21, tail))
                    p(List.cons(step, tail))
                }
                DirectedKonigsbergBridge.e60_23 {
                    directed_incidence_count_v0_cons_e60_23(tail)
                    directed_bridge_count(List.cons(DirectedKonigsbergBridge.e60_23, tail), KonigsbergBridge.e10) = directed_bridge_count(tail, KonigsbergBridge.e10)
                    directed_bridge_count(List.cons(DirectedKonigsbergBridge.e60_23, tail), KonigsbergBridge.e20) = directed_bridge_count(tail, KonigsbergBridge.e20)
                    directed_bridge_count(List.cons(DirectedKonigsbergBridge.e60_23, tail), KonigsbergBridge.e30) = directed_bridge_count(tail, KonigsbergBridge.e30)
                    p(List.cons(DirectedKonigsbergBridge.e60_23, tail))
                    p(List.cons(step, tail))
                }
                DirectedKonigsbergBridge.e60_32 {
                    directed_incidence_count_v0_cons_e60_32(tail)
                    directed_bridge_count(List.cons(DirectedKonigsbergBridge.e60_32, tail), KonigsbergBridge.e10) = directed_bridge_count(tail, KonigsbergBridge.e10)
                    directed_bridge_count(List.cons(DirectedKonigsbergBridge.e60_32, tail), KonigsbergBridge.e20) = directed_bridge_count(tail, KonigsbergBridge.e20)
                    directed_bridge_count(List.cons(DirectedKonigsbergBridge.e60_32, tail), KonigsbergBridge.e30) = directed_bridge_count(tail, KonigsbergBridge.e30)
                    p(List.cons(DirectedKonigsbergBridge.e60_32, tail))
                    p(List.cons(step, tail))
                }
                DirectedKonigsbergBridge.e70_23 {
                    directed_incidence_count_v0_cons_e70_23(tail)
                    directed_bridge_count(List.cons(DirectedKonigsbergBridge.e70_23, tail), KonigsbergBridge.e10) = directed_bridge_count(tail, KonigsbergBridge.e10)
                    directed_bridge_count(List.cons(DirectedKonigsbergBridge.e70_23, tail), KonigsbergBridge.e20) = directed_bridge_count(tail, KonigsbergBridge.e20)
                    directed_bridge_count(List.cons(DirectedKonigsbergBridge.e70_23, tail), KonigsbergBridge.e30) = directed_bridge_count(tail, KonigsbergBridge.e30)
                    p(List.cons(DirectedKonigsbergBridge.e70_23, tail))
                    p(List.cons(step, tail))
                }
                DirectedKonigsbergBridge.e70_32 {
                    directed_incidence_count_v0_cons_e70_32(tail)
                    directed_bridge_count(List.cons(DirectedKonigsbergBridge.e70_32, tail), KonigsbergBridge.e10) = directed_bridge_count(tail, KonigsbergBridge.e10)
                    directed_bridge_count(List.cons(DirectedKonigsbergBridge.e70_32, tail), KonigsbergBridge.e20) = directed_bridge_count(tail, KonigsbergBridge.e20)
                    directed_bridge_count(List.cons(DirectedKonigsbergBridge.e70_32, tail), KonigsbergBridge.e30) = directed_bridge_count(tail, KonigsbergBridge.e30)
                    p(List.cons(DirectedKonigsbergBridge.e70_32, tail))
                    p(List.cons(step, tail))
                }
            }
        }
    }

    p(steps)
}

/// A crossing of bridge `e10` from `v0` to `v1` adds one incident crossing at `v1`.
theorem directed_incidence_count_v1_cons_e10_01(tail: List[DirectedKonigsbergBridge]) {
    directed_incidence_count(List.cons(DirectedKonigsbergBridge.e10_01, tail), KonigsbergVertex.v1) =
        Nat.1 + directed_incidence_count(tail, KonigsbergVertex.v1)
} by {
    directed_incidence_count_cons(DirectedKonigsbergBridge.e10_01, tail, KonigsbergVertex.v1)
    vertex_indicator(directed_source(DirectedKonigsbergBridge.e10_01), KonigsbergVertex.v1) = Nat.0
    vertex_indicator(directed_target(DirectedKonigsbergBridge.e10_01), KonigsbergVertex.v1) = Nat.1
}

/// A crossing of bridge `e10` from `v1` to `v0` adds one incident crossing at `v1`.
theorem directed_incidence_count_v1_cons_e10_10(tail: List[DirectedKonigsbergBridge]) {
    directed_incidence_count(List.cons(DirectedKonigsbergBridge.e10_10, tail), KonigsbergVertex.v1) =
        Nat.1 + directed_incidence_count(tail, KonigsbergVertex.v1)
} by {
    directed_incidence_count_cons(DirectedKonigsbergBridge.e10_10, tail, KonigsbergVertex.v1)
    vertex_indicator(directed_source(DirectedKonigsbergBridge.e10_10), KonigsbergVertex.v1) = Nat.1
    vertex_indicator(directed_target(DirectedKonigsbergBridge.e10_10), KonigsbergVertex.v1) = Nat.0
}

/// A crossing of bridge `e20` does not touch `v1`.
theorem directed_incidence_count_v1_cons_e20_02(tail: List[DirectedKonigsbergBridge]) {
    directed_incidence_count(List.cons(DirectedKonigsbergBridge.e20_02, tail), KonigsbergVertex.v1) =
        directed_incidence_count(tail, KonigsbergVertex.v1)
} by {
    directed_incidence_count_cons(DirectedKonigsbergBridge.e20_02, tail, KonigsbergVertex.v1)
    vertex_indicator(directed_source(DirectedKonigsbergBridge.e20_02), KonigsbergVertex.v1) = Nat.0
    vertex_indicator(directed_target(DirectedKonigsbergBridge.e20_02), KonigsbergVertex.v1) = Nat.0
}

/// A crossing of bridge `e20` does not touch `v1`.
theorem directed_incidence_count_v1_cons_e20_20(tail: List[DirectedKonigsbergBridge]) {
    directed_incidence_count(List.cons(DirectedKonigsbergBridge.e20_20, tail), KonigsbergVertex.v1) =
        directed_incidence_count(tail, KonigsbergVertex.v1)
} by {
    directed_incidence_count_cons(DirectedKonigsbergBridge.e20_20, tail, KonigsbergVertex.v1)
    vertex_indicator(directed_source(DirectedKonigsbergBridge.e20_20), KonigsbergVertex.v1) = Nat.0
    vertex_indicator(directed_target(DirectedKonigsbergBridge.e20_20), KonigsbergVertex.v1) = Nat.0
}

/// A crossing of bridge `e30` does not touch `v1`.
theorem directed_incidence_count_v1_cons_e30_03(tail: List[DirectedKonigsbergBridge]) {
    directed_incidence_count(List.cons(DirectedKonigsbergBridge.e30_03, tail), KonigsbergVertex.v1) =
        directed_incidence_count(tail, KonigsbergVertex.v1)
} by {
    directed_incidence_count_cons(DirectedKonigsbergBridge.e30_03, tail, KonigsbergVertex.v1)
    vertex_indicator(directed_source(DirectedKonigsbergBridge.e30_03), KonigsbergVertex.v1) = Nat.0
    vertex_indicator(directed_target(DirectedKonigsbergBridge.e30_03), KonigsbergVertex.v1) = Nat.0
}

/// A crossing of bridge `e30` does not touch `v1`.
theorem directed_incidence_count_v1_cons_e30_30(tail: List[DirectedKonigsbergBridge]) {
    directed_incidence_count(List.cons(DirectedKonigsbergBridge.e30_30, tail), KonigsbergVertex.v1) =
        directed_incidence_count(tail, KonigsbergVertex.v1)
} by {
    directed_incidence_count_cons(DirectedKonigsbergBridge.e30_30, tail, KonigsbergVertex.v1)
    vertex_indicator(directed_source(DirectedKonigsbergBridge.e30_30), KonigsbergVertex.v1) = Nat.0
    vertex_indicator(directed_target(DirectedKonigsbergBridge.e30_30), KonigsbergVertex.v1) = Nat.0
}

/// A crossing of bridge `e40` from `v1` to `v2` adds one incident crossing at `v1`.
theorem directed_incidence_count_v1_cons_e40_12(tail: List[DirectedKonigsbergBridge]) {
    directed_incidence_count(List.cons(DirectedKonigsbergBridge.e40_12, tail), KonigsbergVertex.v1) =
        Nat.1 + directed_incidence_count(tail, KonigsbergVertex.v1)
} by {
    directed_incidence_count_cons(DirectedKonigsbergBridge.e40_12, tail, KonigsbergVertex.v1)
    vertex_indicator(directed_source(DirectedKonigsbergBridge.e40_12), KonigsbergVertex.v1) = Nat.1
    vertex_indicator(directed_target(DirectedKonigsbergBridge.e40_12), KonigsbergVertex.v1) = Nat.0
}

/// A crossing of bridge `e40` from `v2` to `v1` adds one incident crossing at `v1`.
theorem directed_incidence_count_v1_cons_e40_21(tail: List[DirectedKonigsbergBridge]) {
    directed_incidence_count(List.cons(DirectedKonigsbergBridge.e40_21, tail), KonigsbergVertex.v1) =
        Nat.1 + directed_incidence_count(tail, KonigsbergVertex.v1)
} by {
    directed_incidence_count_cons(DirectedKonigsbergBridge.e40_21, tail, KonigsbergVertex.v1)
    vertex_indicator(directed_source(DirectedKonigsbergBridge.e40_21), KonigsbergVertex.v1) = Nat.0
    vertex_indicator(directed_target(DirectedKonigsbergBridge.e40_21), KonigsbergVertex.v1) = Nat.1
}

/// A crossing of bridge `e50` from `v1` to `v2` adds one incident crossing at `v1`.
theorem directed_incidence_count_v1_cons_e50_12(tail: List[DirectedKonigsbergBridge]) {
    directed_incidence_count(List.cons(DirectedKonigsbergBridge.e50_12, tail), KonigsbergVertex.v1) =
        Nat.1 + directed_incidence_count(tail, KonigsbergVertex.v1)
} by {
    directed_incidence_count_cons(DirectedKonigsbergBridge.e50_12, tail, KonigsbergVertex.v1)
    vertex_indicator(directed_source(DirectedKonigsbergBridge.e50_12), KonigsbergVertex.v1) = Nat.1
    vertex_indicator(directed_target(DirectedKonigsbergBridge.e50_12), KonigsbergVertex.v1) = Nat.0
}

/// A crossing of bridge `e50` from `v2` to `v1` adds one incident crossing at `v1`.
theorem directed_incidence_count_v1_cons_e50_21(tail: List[DirectedKonigsbergBridge]) {
    directed_incidence_count(List.cons(DirectedKonigsbergBridge.e50_21, tail), KonigsbergVertex.v1) =
        Nat.1 + directed_incidence_count(tail, KonigsbergVertex.v1)
} by {
    directed_incidence_count_cons(DirectedKonigsbergBridge.e50_21, tail, KonigsbergVertex.v1)
    vertex_indicator(directed_source(DirectedKonigsbergBridge.e50_21), KonigsbergVertex.v1) = Nat.0
    vertex_indicator(directed_target(DirectedKonigsbergBridge.e50_21), KonigsbergVertex.v1) = Nat.1
}

/// A crossing of bridge `e60` does not touch `v1`.
theorem directed_incidence_count_v1_cons_e60_23(tail: List[DirectedKonigsbergBridge]) {
    directed_incidence_count(List.cons(DirectedKonigsbergBridge.e60_23, tail), KonigsbergVertex.v1) =
        directed_incidence_count(tail, KonigsbergVertex.v1)
} by {
    directed_incidence_count_cons(DirectedKonigsbergBridge.e60_23, tail, KonigsbergVertex.v1)
    vertex_indicator(directed_source(DirectedKonigsbergBridge.e60_23), KonigsbergVertex.v1) = Nat.0
    vertex_indicator(directed_target(DirectedKonigsbergBridge.e60_23), KonigsbergVertex.v1) = Nat.0
}

/// A crossing of bridge `e60` does not touch `v1`.
theorem directed_incidence_count_v1_cons_e60_32(tail: List[DirectedKonigsbergBridge]) {
    directed_incidence_count(List.cons(DirectedKonigsbergBridge.e60_32, tail), KonigsbergVertex.v1) =
        directed_incidence_count(tail, KonigsbergVertex.v1)
} by {
    directed_incidence_count_cons(DirectedKonigsbergBridge.e60_32, tail, KonigsbergVertex.v1)
    vertex_indicator(directed_source(DirectedKonigsbergBridge.e60_32), KonigsbergVertex.v1) = Nat.0
    vertex_indicator(directed_target(DirectedKonigsbergBridge.e60_32), KonigsbergVertex.v1) = Nat.0
}

/// A crossing of bridge `e70` does not touch `v1`.
theorem directed_incidence_count_v1_cons_e70_23(tail: List[DirectedKonigsbergBridge]) {
    directed_incidence_count(List.cons(DirectedKonigsbergBridge.e70_23, tail), KonigsbergVertex.v1) =
        directed_incidence_count(tail, KonigsbergVertex.v1)
} by {
    directed_incidence_count_cons(DirectedKonigsbergBridge.e70_23, tail, KonigsbergVertex.v1)
    vertex_indicator(directed_source(DirectedKonigsbergBridge.e70_23), KonigsbergVertex.v1) = Nat.0
    vertex_indicator(directed_target(DirectedKonigsbergBridge.e70_23), KonigsbergVertex.v1) = Nat.0
}

/// A crossing of bridge `e70` does not touch `v1`.
theorem directed_incidence_count_v1_cons_e70_32(tail: List[DirectedKonigsbergBridge]) {
    directed_incidence_count(List.cons(DirectedKonigsbergBridge.e70_32, tail), KonigsbergVertex.v1) =
        directed_incidence_count(tail, KonigsbergVertex.v1)
} by {
    directed_incidence_count_cons(DirectedKonigsbergBridge.e70_32, tail, KonigsbergVertex.v1)
    vertex_indicator(directed_source(DirectedKonigsbergBridge.e70_32), KonigsbergVertex.v1) = Nat.0
    vertex_indicator(directed_target(DirectedKonigsbergBridge.e70_32), KonigsbergVertex.v1) = Nat.0
}

/// The incident crossings at `v1` are exactly the uses of bridges `e10`, `e40`, and `e50`.
theorem directed_incidence_count_v1(steps: List[DirectedKonigsbergBridge]) {
    directed_incidence_count(steps, KonigsbergVertex.v1) =
        directed_bridge_count(steps, KonigsbergBridge.e10) +
        directed_bridge_count(steps, KonigsbergBridge.e40) +
        directed_bridge_count(steps, KonigsbergBridge.e50)
} by {
    define p(xs: List[DirectedKonigsbergBridge]) -> Bool {
        directed_incidence_count(xs, KonigsbergVertex.v1) =
            directed_bridge_count(xs, KonigsbergBridge.e10) +
            directed_bridge_count(xs, KonigsbergBridge.e40) +
            directed_bridge_count(xs, KonigsbergBridge.e50)
    }

    directed_incidence_count(List.nil[DirectedKonigsbergBridge], KonigsbergVertex.v1) = Nat.0
    directed_bridge_count(List.nil[DirectedKonigsbergBridge], KonigsbergBridge.e10) = Nat.0
    directed_bridge_count(List.nil[DirectedKonigsbergBridge], KonigsbergBridge.e40) = Nat.0
    directed_bridge_count(List.nil[DirectedKonigsbergBridge], KonigsbergBridge.e50) = Nat.0
    Nat.0 = Nat.0 + Nat.0 + Nat.0
    p(List.nil[DirectedKonigsbergBridge])

    forall(step: DirectedKonigsbergBridge, tail: List[DirectedKonigsbergBridge]) {
        if p(tail) {
            match step {
                DirectedKonigsbergBridge.e10_01 {
                    directed_incidence_count_v1_cons_e10_01(tail)
                    let a = directed_bridge_count(tail, KonigsbergBridge.e10)
                    let b = directed_bridge_count(tail, KonigsbergBridge.e40)
                    let c = directed_bridge_count(tail, KonigsbergBridge.e50)
                    directed_incidence_count(tail, KonigsbergVertex.v1) = a + b + c
                    Nat.1 + directed_incidence_count(tail, KonigsbergVertex.v1) = Nat.1 + (a + b + c)
                    Nat.1 + (a + b + c) = Nat.1 + a + b + c
                    directed_bridge_count(List.cons(DirectedKonigsbergBridge.e10_01, tail), KonigsbergBridge.e10) = Nat.1 + directed_bridge_count(tail, KonigsbergBridge.e10)
                    directed_bridge_count(List.cons(DirectedKonigsbergBridge.e10_01, tail), KonigsbergBridge.e40) = directed_bridge_count(tail, KonigsbergBridge.e40)
                    directed_bridge_count(List.cons(DirectedKonigsbergBridge.e10_01, tail), KonigsbergBridge.e50) = directed_bridge_count(tail, KonigsbergBridge.e50)
                    p(List.cons(DirectedKonigsbergBridge.e10_01, tail))
                    p(List.cons(step, tail))
                }
                DirectedKonigsbergBridge.e10_10 {
                    directed_incidence_count_v1_cons_e10_10(tail)
                    let a = directed_bridge_count(tail, KonigsbergBridge.e10)
                    let b = directed_bridge_count(tail, KonigsbergBridge.e40)
                    let c = directed_bridge_count(tail, KonigsbergBridge.e50)
                    directed_incidence_count(tail, KonigsbergVertex.v1) = a + b + c
                    Nat.1 + directed_incidence_count(tail, KonigsbergVertex.v1) = Nat.1 + (a + b + c)
                    Nat.1 + (a + b + c) = Nat.1 + a + b + c
                    directed_bridge_count(List.cons(DirectedKonigsbergBridge.e10_10, tail), KonigsbergBridge.e10) = Nat.1 + directed_bridge_count(tail, KonigsbergBridge.e10)
                    directed_bridge_count(List.cons(DirectedKonigsbergBridge.e10_10, tail), KonigsbergBridge.e40) = directed_bridge_count(tail, KonigsbergBridge.e40)
                    directed_bridge_count(List.cons(DirectedKonigsbergBridge.e10_10, tail), KonigsbergBridge.e50) = directed_bridge_count(tail, KonigsbergBridge.e50)
                    p(List.cons(DirectedKonigsbergBridge.e10_10, tail))
                    p(List.cons(step, tail))
                }
                DirectedKonigsbergBridge.e20_02 {
                    directed_incidence_count_v1_cons_e20_02(tail)
                    directed_bridge_count(List.cons(DirectedKonigsbergBridge.e20_02, tail), KonigsbergBridge.e10) = directed_bridge_count(tail, KonigsbergBridge.e10)
                    directed_bridge_count(List.cons(DirectedKonigsbergBridge.e20_02, tail), KonigsbergBridge.e40) = directed_bridge_count(tail, KonigsbergBridge.e40)
                    directed_bridge_count(List.cons(DirectedKonigsbergBridge.e20_02, tail), KonigsbergBridge.e50) = directed_bridge_count(tail, KonigsbergBridge.e50)
                    p(List.cons(DirectedKonigsbergBridge.e20_02, tail))
                    p(List.cons(step, tail))
                }
                DirectedKonigsbergBridge.e20_20 {
                    directed_incidence_count_v1_cons_e20_20(tail)
                    directed_bridge_count(List.cons(DirectedKonigsbergBridge.e20_20, tail), KonigsbergBridge.e10) = directed_bridge_count(tail, KonigsbergBridge.e10)
                    directed_bridge_count(List.cons(DirectedKonigsbergBridge.e20_20, tail), KonigsbergBridge.e40) = directed_bridge_count(tail, KonigsbergBridge.e40)
                    directed_bridge_count(List.cons(DirectedKonigsbergBridge.e20_20, tail), KonigsbergBridge.e50) = directed_bridge_count(tail, KonigsbergBridge.e50)
                    p(List.cons(DirectedKonigsbergBridge.e20_20, tail))
                    p(List.cons(step, tail))
                }
                DirectedKonigsbergBridge.e30_03 {
                    directed_incidence_count_v1_cons_e30_03(tail)
                    directed_bridge_count(List.cons(DirectedKonigsbergBridge.e30_03, tail), KonigsbergBridge.e10) = directed_bridge_count(tail, KonigsbergBridge.e10)
                    directed_bridge_count(List.cons(DirectedKonigsbergBridge.e30_03, tail), KonigsbergBridge.e40) = directed_bridge_count(tail, KonigsbergBridge.e40)
                    directed_bridge_count(List.cons(DirectedKonigsbergBridge.e30_03, tail), KonigsbergBridge.e50) = directed_bridge_count(tail, KonigsbergBridge.e50)
                    p(List.cons(DirectedKonigsbergBridge.e30_03, tail))
                    p(List.cons(step, tail))
                }
                DirectedKonigsbergBridge.e30_30 {
                    directed_incidence_count_v1_cons_e30_30(tail)
                    directed_bridge_count(List.cons(DirectedKonigsbergBridge.e30_30, tail), KonigsbergBridge.e10) = directed_bridge_count(tail, KonigsbergBridge.e10)
                    directed_bridge_count(List.cons(DirectedKonigsbergBridge.e30_30, tail), KonigsbergBridge.e40) = directed_bridge_count(tail, KonigsbergBridge.e40)
                    directed_bridge_count(List.cons(DirectedKonigsbergBridge.e30_30, tail), KonigsbergBridge.e50) = directed_bridge_count(tail, KonigsbergBridge.e50)
                    p(List.cons(DirectedKonigsbergBridge.e30_30, tail))
                    p(List.cons(step, tail))
                }
                DirectedKonigsbergBridge.e40_12 {
                    directed_incidence_count_v1_cons_e40_12(tail)
                    let a = directed_bridge_count(tail, KonigsbergBridge.e10)
                    let b = directed_bridge_count(tail, KonigsbergBridge.e40)
                    let c = directed_bridge_count(tail, KonigsbergBridge.e50)
                    directed_incidence_count(tail, KonigsbergVertex.v1) = a + b + c
                    Nat.1 + directed_incidence_count(tail, KonigsbergVertex.v1) = Nat.1 + (a + b + c)
                    Nat.1 + (a + b + c) = Nat.1 + a + b + c
                    Nat.1 + a = a + Nat.1
                    Nat.1 + a + b + c = a + Nat.1 + b + c
                    a + Nat.1 + b + c = a + (Nat.1 + b) + c
                    directed_bridge_count(List.cons(DirectedKonigsbergBridge.e40_12, tail), KonigsbergBridge.e10) = directed_bridge_count(tail, KonigsbergBridge.e10)
                    directed_bridge_count(List.cons(DirectedKonigsbergBridge.e40_12, tail), KonigsbergBridge.e40) = Nat.1 + directed_bridge_count(tail, KonigsbergBridge.e40)
                    directed_bridge_count(List.cons(DirectedKonigsbergBridge.e40_12, tail), KonigsbergBridge.e50) = directed_bridge_count(tail, KonigsbergBridge.e50)
                    p(List.cons(DirectedKonigsbergBridge.e40_12, tail))
                    p(List.cons(step, tail))
                }
                DirectedKonigsbergBridge.e40_21 {
                    directed_incidence_count_v1_cons_e40_21(tail)
                    let a = directed_bridge_count(tail, KonigsbergBridge.e10)
                    let b = directed_bridge_count(tail, KonigsbergBridge.e40)
                    let c = directed_bridge_count(tail, KonigsbergBridge.e50)
                    directed_incidence_count(tail, KonigsbergVertex.v1) = a + b + c
                    Nat.1 + directed_incidence_count(tail, KonigsbergVertex.v1) = Nat.1 + (a + b + c)
                    Nat.1 + (a + b + c) = Nat.1 + a + b + c
                    Nat.1 + a = a + Nat.1
                    Nat.1 + a + b + c = a + Nat.1 + b + c
                    a + Nat.1 + b + c = a + (Nat.1 + b) + c
                    directed_bridge_count(List.cons(DirectedKonigsbergBridge.e40_21, tail), KonigsbergBridge.e10) = directed_bridge_count(tail, KonigsbergBridge.e10)
                    directed_bridge_count(List.cons(DirectedKonigsbergBridge.e40_21, tail), KonigsbergBridge.e40) = Nat.1 + directed_bridge_count(tail, KonigsbergBridge.e40)
                    directed_bridge_count(List.cons(DirectedKonigsbergBridge.e40_21, tail), KonigsbergBridge.e50) = directed_bridge_count(tail, KonigsbergBridge.e50)
                    p(List.cons(DirectedKonigsbergBridge.e40_21, tail))
                    p(List.cons(step, tail))
                }
                DirectedKonigsbergBridge.e50_12 {
                    directed_incidence_count_v1_cons_e50_12(tail)
                    let a = directed_bridge_count(tail, KonigsbergBridge.e10)
                    let b = directed_bridge_count(tail, KonigsbergBridge.e40)
                    let c = directed_bridge_count(tail, KonigsbergBridge.e50)
                    directed_incidence_count(tail, KonigsbergVertex.v1) = a + b + c
                    Nat.1 + directed_incidence_count(tail, KonigsbergVertex.v1) = Nat.1 + (a + b + c)
                    Nat.1 + (a + b + c) = Nat.1 + a + b + c
                    Nat.1 + a = a + Nat.1
                    Nat.1 + a + b + c = a + Nat.1 + b + c
                    Nat.1 + b = b + Nat.1
                    a + Nat.1 + b + c = a + b + Nat.1 + c
                    a + b + Nat.1 + c = a + b + (Nat.1 + c)
                    directed_bridge_count(List.cons(DirectedKonigsbergBridge.e50_12, tail), KonigsbergBridge.e10) = directed_bridge_count(tail, KonigsbergBridge.e10)
                    directed_bridge_count(List.cons(DirectedKonigsbergBridge.e50_12, tail), KonigsbergBridge.e40) = directed_bridge_count(tail, KonigsbergBridge.e40)
                    directed_bridge_count(List.cons(DirectedKonigsbergBridge.e50_12, tail), KonigsbergBridge.e50) = Nat.1 + directed_bridge_count(tail, KonigsbergBridge.e50)
                    p(List.cons(DirectedKonigsbergBridge.e50_12, tail))
                    p(List.cons(step, tail))
                }
                DirectedKonigsbergBridge.e50_21 {
                    directed_incidence_count_v1_cons_e50_21(tail)
                    let a = directed_bridge_count(tail, KonigsbergBridge.e10)
                    let b = directed_bridge_count(tail, KonigsbergBridge.e40)
                    let c = directed_bridge_count(tail, KonigsbergBridge.e50)
                    directed_incidence_count(tail, KonigsbergVertex.v1) = a + b + c
                    Nat.1 + directed_incidence_count(tail, KonigsbergVertex.v1) = Nat.1 + (a + b + c)
                    Nat.1 + (a + b + c) = Nat.1 + a + b + c
                    Nat.1 + a = a + Nat.1
                    Nat.1 + a + b + c = a + Nat.1 + b + c
                    Nat.1 + b = b + Nat.1
                    a + Nat.1 + b + c = a + b + Nat.1 + c
                    a + b + Nat.1 + c = a + b + (Nat.1 + c)
                    directed_bridge_count(List.cons(DirectedKonigsbergBridge.e50_21, tail), KonigsbergBridge.e10) = directed_bridge_count(tail, KonigsbergBridge.e10)
                    directed_bridge_count(List.cons(DirectedKonigsbergBridge.e50_21, tail), KonigsbergBridge.e40) = directed_bridge_count(tail, KonigsbergBridge.e40)
                    directed_bridge_count(List.cons(DirectedKonigsbergBridge.e50_21, tail), KonigsbergBridge.e50) = Nat.1 + directed_bridge_count(tail, KonigsbergBridge.e50)
                    p(List.cons(DirectedKonigsbergBridge.e50_21, tail))
                    p(List.cons(step, tail))
                }
                DirectedKonigsbergBridge.e60_23 {
                    directed_incidence_count_v1_cons_e60_23(tail)
                    directed_bridge_count(List.cons(DirectedKonigsbergBridge.e60_23, tail), KonigsbergBridge.e10) = directed_bridge_count(tail, KonigsbergBridge.e10)
                    directed_bridge_count(List.cons(DirectedKonigsbergBridge.e60_23, tail), KonigsbergBridge.e40) = directed_bridge_count(tail, KonigsbergBridge.e40)
                    directed_bridge_count(List.cons(DirectedKonigsbergBridge.e60_23, tail), KonigsbergBridge.e50) = directed_bridge_count(tail, KonigsbergBridge.e50)
                    p(List.cons(DirectedKonigsbergBridge.e60_23, tail))
                    p(List.cons(step, tail))
                }
                DirectedKonigsbergBridge.e60_32 {
                    directed_incidence_count_v1_cons_e60_32(tail)
                    directed_bridge_count(List.cons(DirectedKonigsbergBridge.e60_32, tail), KonigsbergBridge.e10) = directed_bridge_count(tail, KonigsbergBridge.e10)
                    directed_bridge_count(List.cons(DirectedKonigsbergBridge.e60_32, tail), KonigsbergBridge.e40) = directed_bridge_count(tail, KonigsbergBridge.e40)
                    directed_bridge_count(List.cons(DirectedKonigsbergBridge.e60_32, tail), KonigsbergBridge.e50) = directed_bridge_count(tail, KonigsbergBridge.e50)
                    p(List.cons(DirectedKonigsbergBridge.e60_32, tail))
                    p(List.cons(step, tail))
                }
                DirectedKonigsbergBridge.e70_23 {
                    directed_incidence_count_v1_cons_e70_23(tail)
                    directed_bridge_count(List.cons(DirectedKonigsbergBridge.e70_23, tail), KonigsbergBridge.e10) = directed_bridge_count(tail, KonigsbergBridge.e10)
                    directed_bridge_count(List.cons(DirectedKonigsbergBridge.e70_23, tail), KonigsbergBridge.e40) = directed_bridge_count(tail, KonigsbergBridge.e40)
                    directed_bridge_count(List.cons(DirectedKonigsbergBridge.e70_23, tail), KonigsbergBridge.e50) = directed_bridge_count(tail, KonigsbergBridge.e50)
                    p(List.cons(DirectedKonigsbergBridge.e70_23, tail))
                    p(List.cons(step, tail))
                }
                DirectedKonigsbergBridge.e70_32 {
                    directed_incidence_count_v1_cons_e70_32(tail)
                    directed_bridge_count(List.cons(DirectedKonigsbergBridge.e70_32, tail), KonigsbergBridge.e10) = directed_bridge_count(tail, KonigsbergBridge.e10)
                    directed_bridge_count(List.cons(DirectedKonigsbergBridge.e70_32, tail), KonigsbergBridge.e40) = directed_bridge_count(tail, KonigsbergBridge.e40)
                    directed_bridge_count(List.cons(DirectedKonigsbergBridge.e70_32, tail), KonigsbergBridge.e50) = directed_bridge_count(tail, KonigsbergBridge.e50)
                    p(List.cons(DirectedKonigsbergBridge.e70_32, tail))
                    p(List.cons(step, tail))
                }
            }
        }
    }

    p(steps)
}

/// A crossing of bridge `e10` does not touch `v2`.
theorem directed_incidence_count_v2_cons_e10_01(tail: List[DirectedKonigsbergBridge]) {
    directed_incidence_count(List.cons(DirectedKonigsbergBridge.e10_01, tail), KonigsbergVertex.v2) =
        directed_incidence_count(tail, KonigsbergVertex.v2)
} by {
    directed_incidence_count_cons(DirectedKonigsbergBridge.e10_01, tail, KonigsbergVertex.v2)
    vertex_indicator(directed_source(DirectedKonigsbergBridge.e10_01), KonigsbergVertex.v2) = Nat.0
    vertex_indicator(directed_target(DirectedKonigsbergBridge.e10_01), KonigsbergVertex.v2) = Nat.0
}

/// A crossing of bridge `e10` does not touch `v2`.
theorem directed_incidence_count_v2_cons_e10_10(tail: List[DirectedKonigsbergBridge]) {
    directed_incidence_count(List.cons(DirectedKonigsbergBridge.e10_10, tail), KonigsbergVertex.v2) =
        directed_incidence_count(tail, KonigsbergVertex.v2)
} by {
    directed_incidence_count_cons(DirectedKonigsbergBridge.e10_10, tail, KonigsbergVertex.v2)
    vertex_indicator(directed_source(DirectedKonigsbergBridge.e10_10), KonigsbergVertex.v2) = Nat.0
    vertex_indicator(directed_target(DirectedKonigsbergBridge.e10_10), KonigsbergVertex.v2) = Nat.0
}

/// A crossing of bridge `e20` from `v0` to `v2` adds one incident crossing at `v2`.
theorem directed_incidence_count_v2_cons_e20_02(tail: List[DirectedKonigsbergBridge]) {
    directed_incidence_count(List.cons(DirectedKonigsbergBridge.e20_02, tail), KonigsbergVertex.v2) =
        Nat.1 + directed_incidence_count(tail, KonigsbergVertex.v2)
} by {
    directed_incidence_count_cons(DirectedKonigsbergBridge.e20_02, tail, KonigsbergVertex.v2)
    vertex_indicator(directed_source(DirectedKonigsbergBridge.e20_02), KonigsbergVertex.v2) = Nat.0
    vertex_indicator(directed_target(DirectedKonigsbergBridge.e20_02), KonigsbergVertex.v2) = Nat.1
}

/// A crossing of bridge `e20` from `v2` to `v0` adds one incident crossing at `v2`.
theorem directed_incidence_count_v2_cons_e20_20(tail: List[DirectedKonigsbergBridge]) {
    directed_incidence_count(List.cons(DirectedKonigsbergBridge.e20_20, tail), KonigsbergVertex.v2) =
        Nat.1 + directed_incidence_count(tail, KonigsbergVertex.v2)
} by {
    directed_incidence_count_cons(DirectedKonigsbergBridge.e20_20, tail, KonigsbergVertex.v2)
    vertex_indicator(directed_source(DirectedKonigsbergBridge.e20_20), KonigsbergVertex.v2) = Nat.1
    vertex_indicator(directed_target(DirectedKonigsbergBridge.e20_20), KonigsbergVertex.v2) = Nat.0
}

/// A crossing of bridge `e30` does not touch `v2`.
theorem directed_incidence_count_v2_cons_e30_03(tail: List[DirectedKonigsbergBridge]) {
    directed_incidence_count(List.cons(DirectedKonigsbergBridge.e30_03, tail), KonigsbergVertex.v2) =
        directed_incidence_count(tail, KonigsbergVertex.v2)
} by {
    directed_incidence_count_cons(DirectedKonigsbergBridge.e30_03, tail, KonigsbergVertex.v2)
    vertex_indicator(directed_source(DirectedKonigsbergBridge.e30_03), KonigsbergVertex.v2) = Nat.0
    vertex_indicator(directed_target(DirectedKonigsbergBridge.e30_03), KonigsbergVertex.v2) = Nat.0
}

/// A crossing of bridge `e30` does not touch `v2`.
theorem directed_incidence_count_v2_cons_e30_30(tail: List[DirectedKonigsbergBridge]) {
    directed_incidence_count(List.cons(DirectedKonigsbergBridge.e30_30, tail), KonigsbergVertex.v2) =
        directed_incidence_count(tail, KonigsbergVertex.v2)
} by {
    directed_incidence_count_cons(DirectedKonigsbergBridge.e30_30, tail, KonigsbergVertex.v2)
    vertex_indicator(directed_source(DirectedKonigsbergBridge.e30_30), KonigsbergVertex.v2) = Nat.0
    vertex_indicator(directed_target(DirectedKonigsbergBridge.e30_30), KonigsbergVertex.v2) = Nat.0
}

/// A crossing of bridge `e40` from `v1` to `v2` adds one incident crossing at `v2`.
theorem directed_incidence_count_v2_cons_e40_12(tail: List[DirectedKonigsbergBridge]) {
    directed_incidence_count(List.cons(DirectedKonigsbergBridge.e40_12, tail), KonigsbergVertex.v2) =
        Nat.1 + directed_incidence_count(tail, KonigsbergVertex.v2)
} by {
    directed_incidence_count_cons(DirectedKonigsbergBridge.e40_12, tail, KonigsbergVertex.v2)
    vertex_indicator(directed_source(DirectedKonigsbergBridge.e40_12), KonigsbergVertex.v2) = Nat.0
    vertex_indicator(directed_target(DirectedKonigsbergBridge.e40_12), KonigsbergVertex.v2) = Nat.1
}

/// A crossing of bridge `e40` from `v2` to `v1` adds one incident crossing at `v2`.
theorem directed_incidence_count_v2_cons_e40_21(tail: List[DirectedKonigsbergBridge]) {
    directed_incidence_count(List.cons(DirectedKonigsbergBridge.e40_21, tail), KonigsbergVertex.v2) =
        Nat.1 + directed_incidence_count(tail, KonigsbergVertex.v2)
} by {
    directed_incidence_count_cons(DirectedKonigsbergBridge.e40_21, tail, KonigsbergVertex.v2)
    vertex_indicator(directed_source(DirectedKonigsbergBridge.e40_21), KonigsbergVertex.v2) = Nat.1
    vertex_indicator(directed_target(DirectedKonigsbergBridge.e40_21), KonigsbergVertex.v2) = Nat.0
}

/// A crossing of bridge `e50` from `v1` to `v2` adds one incident crossing at `v2`.
theorem directed_incidence_count_v2_cons_e50_12(tail: List[DirectedKonigsbergBridge]) {
    directed_incidence_count(List.cons(DirectedKonigsbergBridge.e50_12, tail), KonigsbergVertex.v2) =
        Nat.1 + directed_incidence_count(tail, KonigsbergVertex.v2)
} by {
    directed_incidence_count_cons(DirectedKonigsbergBridge.e50_12, tail, KonigsbergVertex.v2)
    vertex_indicator(directed_source(DirectedKonigsbergBridge.e50_12), KonigsbergVertex.v2) = Nat.0
    vertex_indicator(directed_target(DirectedKonigsbergBridge.e50_12), KonigsbergVertex.v2) = Nat.1
}

/// A crossing of bridge `e50` from `v2` to `v1` adds one incident crossing at `v2`.
theorem directed_incidence_count_v2_cons_e50_21(tail: List[DirectedKonigsbergBridge]) {
    directed_incidence_count(List.cons(DirectedKonigsbergBridge.e50_21, tail), KonigsbergVertex.v2) =
        Nat.1 + directed_incidence_count(tail, KonigsbergVertex.v2)
} by {
    directed_incidence_count_cons(DirectedKonigsbergBridge.e50_21, tail, KonigsbergVertex.v2)
    vertex_indicator(directed_source(DirectedKonigsbergBridge.e50_21), KonigsbergVertex.v2) = Nat.1
    vertex_indicator(directed_target(DirectedKonigsbergBridge.e50_21), KonigsbergVertex.v2) = Nat.0
}

/// A crossing of bridge `e60` from `v2` to `v3` adds one incident crossing at `v2`.
theorem directed_incidence_count_v2_cons_e60_23(tail: List[DirectedKonigsbergBridge]) {
    directed_incidence_count(List.cons(DirectedKonigsbergBridge.e60_23, tail), KonigsbergVertex.v2) =
        Nat.1 + directed_incidence_count(tail, KonigsbergVertex.v2)
} by {
    directed_incidence_count_cons(DirectedKonigsbergBridge.e60_23, tail, KonigsbergVertex.v2)
    vertex_indicator(directed_source(DirectedKonigsbergBridge.e60_23), KonigsbergVertex.v2) = Nat.1
    vertex_indicator(directed_target(DirectedKonigsbergBridge.e60_23), KonigsbergVertex.v2) = Nat.0
}

/// A crossing of bridge `e60` from `v3` to `v2` adds one incident crossing at `v2`.
theorem directed_incidence_count_v2_cons_e60_32(tail: List[DirectedKonigsbergBridge]) {
    directed_incidence_count(List.cons(DirectedKonigsbergBridge.e60_32, tail), KonigsbergVertex.v2) =
        Nat.1 + directed_incidence_count(tail, KonigsbergVertex.v2)
} by {
    directed_incidence_count_cons(DirectedKonigsbergBridge.e60_32, tail, KonigsbergVertex.v2)
    vertex_indicator(directed_source(DirectedKonigsbergBridge.e60_32), KonigsbergVertex.v2) = Nat.0
    vertex_indicator(directed_target(DirectedKonigsbergBridge.e60_32), KonigsbergVertex.v2) = Nat.1
}

/// A crossing of bridge `e70` from `v2` to `v3` adds one incident crossing at `v2`.
theorem directed_incidence_count_v2_cons_e70_23(tail: List[DirectedKonigsbergBridge]) {
    directed_incidence_count(List.cons(DirectedKonigsbergBridge.e70_23, tail), KonigsbergVertex.v2) =
        Nat.1 + directed_incidence_count(tail, KonigsbergVertex.v2)
} by {
    directed_incidence_count_cons(DirectedKonigsbergBridge.e70_23, tail, KonigsbergVertex.v2)
    vertex_indicator(directed_source(DirectedKonigsbergBridge.e70_23), KonigsbergVertex.v2) = Nat.1
    vertex_indicator(directed_target(DirectedKonigsbergBridge.e70_23), KonigsbergVertex.v2) = Nat.0
}

/// A crossing of bridge `e70` from `v3` to `v2` adds one incident crossing at `v2`.
theorem directed_incidence_count_v2_cons_e70_32(tail: List[DirectedKonigsbergBridge]) {
    directed_incidence_count(List.cons(DirectedKonigsbergBridge.e70_32, tail), KonigsbergVertex.v2) =
        Nat.1 + directed_incidence_count(tail, KonigsbergVertex.v2)
} by {
    directed_incidence_count_cons(DirectedKonigsbergBridge.e70_32, tail, KonigsbergVertex.v2)
    vertex_indicator(directed_source(DirectedKonigsbergBridge.e70_32), KonigsbergVertex.v2) = Nat.0
    vertex_indicator(directed_target(DirectedKonigsbergBridge.e70_32), KonigsbergVertex.v2) = Nat.1
}

/// The incident crossings at `v2` are exactly the uses of bridges `e20`, `e40`, `e50`, `e60`, and `e70`.
theorem directed_incidence_count_v2(steps: List[DirectedKonigsbergBridge]) {
    directed_incidence_count(steps, KonigsbergVertex.v2) =
        directed_bridge_count(steps, KonigsbergBridge.e20) +
        directed_bridge_count(steps, KonigsbergBridge.e40) +
        directed_bridge_count(steps, KonigsbergBridge.e50) +
        directed_bridge_count(steps, KonigsbergBridge.e60) +
        directed_bridge_count(steps, KonigsbergBridge.e70)
} by {
    define p(xs: List[DirectedKonigsbergBridge]) -> Bool {
        directed_incidence_count(xs, KonigsbergVertex.v2) =
            directed_bridge_count(xs, KonigsbergBridge.e20) +
            directed_bridge_count(xs, KonigsbergBridge.e40) +
            directed_bridge_count(xs, KonigsbergBridge.e50) +
            directed_bridge_count(xs, KonigsbergBridge.e60) +
            directed_bridge_count(xs, KonigsbergBridge.e70)
    }

    directed_incidence_count(List.nil[DirectedKonigsbergBridge], KonigsbergVertex.v2) = Nat.0
    directed_bridge_count(List.nil[DirectedKonigsbergBridge], KonigsbergBridge.e20) = Nat.0
    directed_bridge_count(List.nil[DirectedKonigsbergBridge], KonigsbergBridge.e40) = Nat.0
    directed_bridge_count(List.nil[DirectedKonigsbergBridge], KonigsbergBridge.e50) = Nat.0
    directed_bridge_count(List.nil[DirectedKonigsbergBridge], KonigsbergBridge.e60) = Nat.0
    directed_bridge_count(List.nil[DirectedKonigsbergBridge], KonigsbergBridge.e70) = Nat.0
    Nat.0 = Nat.0 + Nat.0 + Nat.0 + Nat.0 + Nat.0
    p(List.nil[DirectedKonigsbergBridge])

    forall(step: DirectedKonigsbergBridge, tail: List[DirectedKonigsbergBridge]) {
        if p(tail) {
            match step {
                DirectedKonigsbergBridge.e10_01 {
                    directed_incidence_count_v2_cons_e10_01(tail)
                    directed_bridge_count(List.cons(DirectedKonigsbergBridge.e10_01, tail), KonigsbergBridge.e20) = directed_bridge_count(tail, KonigsbergBridge.e20)
                    directed_bridge_count(List.cons(DirectedKonigsbergBridge.e10_01, tail), KonigsbergBridge.e40) = directed_bridge_count(tail, KonigsbergBridge.e40)
                    directed_bridge_count(List.cons(DirectedKonigsbergBridge.e10_01, tail), KonigsbergBridge.e50) = directed_bridge_count(tail, KonigsbergBridge.e50)
                    directed_bridge_count(List.cons(DirectedKonigsbergBridge.e10_01, tail), KonigsbergBridge.e60) = directed_bridge_count(tail, KonigsbergBridge.e60)
                    directed_bridge_count(List.cons(DirectedKonigsbergBridge.e10_01, tail), KonigsbergBridge.e70) = directed_bridge_count(tail, KonigsbergBridge.e70)
                    p(List.cons(DirectedKonigsbergBridge.e10_01, tail))
                    p(List.cons(step, tail))
                }
                DirectedKonigsbergBridge.e10_10 {
                    directed_incidence_count_v2_cons_e10_10(tail)
                    directed_bridge_count(List.cons(DirectedKonigsbergBridge.e10_10, tail), KonigsbergBridge.e20) = directed_bridge_count(tail, KonigsbergBridge.e20)
                    directed_bridge_count(List.cons(DirectedKonigsbergBridge.e10_10, tail), KonigsbergBridge.e40) = directed_bridge_count(tail, KonigsbergBridge.e40)
                    directed_bridge_count(List.cons(DirectedKonigsbergBridge.e10_10, tail), KonigsbergBridge.e50) = directed_bridge_count(tail, KonigsbergBridge.e50)
                    directed_bridge_count(List.cons(DirectedKonigsbergBridge.e10_10, tail), KonigsbergBridge.e60) = directed_bridge_count(tail, KonigsbergBridge.e60)
                    directed_bridge_count(List.cons(DirectedKonigsbergBridge.e10_10, tail), KonigsbergBridge.e70) = directed_bridge_count(tail, KonigsbergBridge.e70)
                    p(List.cons(DirectedKonigsbergBridge.e10_10, tail))
                    p(List.cons(step, tail))
                }
                DirectedKonigsbergBridge.e20_02 {
                    directed_incidence_count_v2_cons_e20_02(tail)
                    let a = directed_bridge_count(tail, KonigsbergBridge.e20)
                    let b = directed_bridge_count(tail, KonigsbergBridge.e40)
                    let c = directed_bridge_count(tail, KonigsbergBridge.e50)
                    let d = directed_bridge_count(tail, KonigsbergBridge.e60)
                    let e = directed_bridge_count(tail, KonigsbergBridge.e70)
                    directed_incidence_count(tail, KonigsbergVertex.v2) = a + b + c + d + e
                    Nat.1 + directed_incidence_count(tail, KonigsbergVertex.v2) = Nat.1 + (a + b + c + d + e)
                    nat_one_add_sum5(a, b, c, d, e)
                    directed_bridge_count(List.cons(DirectedKonigsbergBridge.e20_02, tail), KonigsbergBridge.e20) = Nat.1 + directed_bridge_count(tail, KonigsbergBridge.e20)
                    directed_bridge_count(List.cons(DirectedKonigsbergBridge.e20_02, tail), KonigsbergBridge.e40) = directed_bridge_count(tail, KonigsbergBridge.e40)
                    directed_bridge_count(List.cons(DirectedKonigsbergBridge.e20_02, tail), KonigsbergBridge.e50) = directed_bridge_count(tail, KonigsbergBridge.e50)
                    directed_bridge_count(List.cons(DirectedKonigsbergBridge.e20_02, tail), KonigsbergBridge.e60) = directed_bridge_count(tail, KonigsbergBridge.e60)
                    directed_bridge_count(List.cons(DirectedKonigsbergBridge.e20_02, tail), KonigsbergBridge.e70) = directed_bridge_count(tail, KonigsbergBridge.e70)
                    p(List.cons(DirectedKonigsbergBridge.e20_02, tail))
                    p(List.cons(step, tail))
                }
                DirectedKonigsbergBridge.e20_20 {
                    directed_incidence_count_v2_cons_e20_20(tail)
                    let a = directed_bridge_count(tail, KonigsbergBridge.e20)
                    let b = directed_bridge_count(tail, KonigsbergBridge.e40)
                    let c = directed_bridge_count(tail, KonigsbergBridge.e50)
                    let d = directed_bridge_count(tail, KonigsbergBridge.e60)
                    let e = directed_bridge_count(tail, KonigsbergBridge.e70)
                    directed_incidence_count(tail, KonigsbergVertex.v2) = a + b + c + d + e
                    Nat.1 + directed_incidence_count(tail, KonigsbergVertex.v2) = Nat.1 + (a + b + c + d + e)
                    nat_one_add_sum5(a, b, c, d, e)
                    directed_bridge_count(List.cons(DirectedKonigsbergBridge.e20_20, tail), KonigsbergBridge.e20) = Nat.1 + directed_bridge_count(tail, KonigsbergBridge.e20)
                    directed_bridge_count(List.cons(DirectedKonigsbergBridge.e20_20, tail), KonigsbergBridge.e40) = directed_bridge_count(tail, KonigsbergBridge.e40)
                    directed_bridge_count(List.cons(DirectedKonigsbergBridge.e20_20, tail), KonigsbergBridge.e50) = directed_bridge_count(tail, KonigsbergBridge.e50)
                    directed_bridge_count(List.cons(DirectedKonigsbergBridge.e20_20, tail), KonigsbergBridge.e60) = directed_bridge_count(tail, KonigsbergBridge.e60)
                    directed_bridge_count(List.cons(DirectedKonigsbergBridge.e20_20, tail), KonigsbergBridge.e70) = directed_bridge_count(tail, KonigsbergBridge.e70)
                    p(List.cons(DirectedKonigsbergBridge.e20_20, tail))
                    p(List.cons(step, tail))
                }
                DirectedKonigsbergBridge.e30_03 {
                    directed_incidence_count_v2_cons_e30_03(tail)
                    directed_bridge_count(List.cons(DirectedKonigsbergBridge.e30_03, tail), KonigsbergBridge.e20) = directed_bridge_count(tail, KonigsbergBridge.e20)
                    directed_bridge_count(List.cons(DirectedKonigsbergBridge.e30_03, tail), KonigsbergBridge.e40) = directed_bridge_count(tail, KonigsbergBridge.e40)
                    directed_bridge_count(List.cons(DirectedKonigsbergBridge.e30_03, tail), KonigsbergBridge.e50) = directed_bridge_count(tail, KonigsbergBridge.e50)
                    directed_bridge_count(List.cons(DirectedKonigsbergBridge.e30_03, tail), KonigsbergBridge.e60) = directed_bridge_count(tail, KonigsbergBridge.e60)
                    directed_bridge_count(List.cons(DirectedKonigsbergBridge.e30_03, tail), KonigsbergBridge.e70) = directed_bridge_count(tail, KonigsbergBridge.e70)
                    p(List.cons(DirectedKonigsbergBridge.e30_03, tail))
                    p(List.cons(step, tail))
                }
                DirectedKonigsbergBridge.e30_30 {
                    directed_incidence_count_v2_cons_e30_30(tail)
                    directed_bridge_count(List.cons(DirectedKonigsbergBridge.e30_30, tail), KonigsbergBridge.e20) = directed_bridge_count(tail, KonigsbergBridge.e20)
                    directed_bridge_count(List.cons(DirectedKonigsbergBridge.e30_30, tail), KonigsbergBridge.e40) = directed_bridge_count(tail, KonigsbergBridge.e40)
                    directed_bridge_count(List.cons(DirectedKonigsbergBridge.e30_30, tail), KonigsbergBridge.e50) = directed_bridge_count(tail, KonigsbergBridge.e50)
                    directed_bridge_count(List.cons(DirectedKonigsbergBridge.e30_30, tail), KonigsbergBridge.e60) = directed_bridge_count(tail, KonigsbergBridge.e60)
                    directed_bridge_count(List.cons(DirectedKonigsbergBridge.e30_30, tail), KonigsbergBridge.e70) = directed_bridge_count(tail, KonigsbergBridge.e70)
                    p(List.cons(DirectedKonigsbergBridge.e30_30, tail))
                    p(List.cons(step, tail))
                }
                DirectedKonigsbergBridge.e40_12 {
                    directed_incidence_count_v2_cons_e40_12(tail)
                    let a = directed_bridge_count(tail, KonigsbergBridge.e20)
                    let b = directed_bridge_count(tail, KonigsbergBridge.e40)
                    let c = directed_bridge_count(tail, KonigsbergBridge.e50)
                    let d = directed_bridge_count(tail, KonigsbergBridge.e60)
                    let e = directed_bridge_count(tail, KonigsbergBridge.e70)
                    directed_incidence_count(tail, KonigsbergVertex.v2) = a + b + c + d + e
                    Nat.1 + directed_incidence_count(tail, KonigsbergVertex.v2) = Nat.1 + (a + b + c + d + e)
                    nat_one_add_sum5(a, b, c, d, e)
                    Nat.1 + a = a + Nat.1
                    Nat.1 + a + b + c + d + e = a + Nat.1 + b + c + d + e
                    a + Nat.1 + b + c + d + e = a + (Nat.1 + b) + c + d + e
                    directed_bridge_count(List.cons(DirectedKonigsbergBridge.e40_12, tail), KonigsbergBridge.e20) = directed_bridge_count(tail, KonigsbergBridge.e20)
                    directed_bridge_count(List.cons(DirectedKonigsbergBridge.e40_12, tail), KonigsbergBridge.e40) = Nat.1 + directed_bridge_count(tail, KonigsbergBridge.e40)
                    directed_bridge_count(List.cons(DirectedKonigsbergBridge.e40_12, tail), KonigsbergBridge.e50) = directed_bridge_count(tail, KonigsbergBridge.e50)
                    directed_bridge_count(List.cons(DirectedKonigsbergBridge.e40_12, tail), KonigsbergBridge.e60) = directed_bridge_count(tail, KonigsbergBridge.e60)
                    directed_bridge_count(List.cons(DirectedKonigsbergBridge.e40_12, tail), KonigsbergBridge.e70) = directed_bridge_count(tail, KonigsbergBridge.e70)
                    p(List.cons(DirectedKonigsbergBridge.e40_12, tail))
                    p(List.cons(step, tail))
                }
                DirectedKonigsbergBridge.e40_21 {
                    directed_incidence_count_v2_cons_e40_21(tail)
                    let a = directed_bridge_count(tail, KonigsbergBridge.e20)
                    let b = directed_bridge_count(tail, KonigsbergBridge.e40)
                    let c = directed_bridge_count(tail, KonigsbergBridge.e50)
                    let d = directed_bridge_count(tail, KonigsbergBridge.e60)
                    let e = directed_bridge_count(tail, KonigsbergBridge.e70)
                    directed_incidence_count(tail, KonigsbergVertex.v2) = a + b + c + d + e
                    Nat.1 + directed_incidence_count(tail, KonigsbergVertex.v2) = Nat.1 + (a + b + c + d + e)
                    nat_one_add_sum5(a, b, c, d, e)
                    Nat.1 + a = a + Nat.1
                    Nat.1 + a + b + c + d + e = a + Nat.1 + b + c + d + e
                    a + Nat.1 + b + c + d + e = a + (Nat.1 + b) + c + d + e
                    directed_bridge_count(List.cons(DirectedKonigsbergBridge.e40_21, tail), KonigsbergBridge.e20) = directed_bridge_count(tail, KonigsbergBridge.e20)
                    directed_bridge_count(List.cons(DirectedKonigsbergBridge.e40_21, tail), KonigsbergBridge.e40) = Nat.1 + directed_bridge_count(tail, KonigsbergBridge.e40)
                    directed_bridge_count(List.cons(DirectedKonigsbergBridge.e40_21, tail), KonigsbergBridge.e50) = directed_bridge_count(tail, KonigsbergBridge.e50)
                    directed_bridge_count(List.cons(DirectedKonigsbergBridge.e40_21, tail), KonigsbergBridge.e60) = directed_bridge_count(tail, KonigsbergBridge.e60)
                    directed_bridge_count(List.cons(DirectedKonigsbergBridge.e40_21, tail), KonigsbergBridge.e70) = directed_bridge_count(tail, KonigsbergBridge.e70)
                    p(List.cons(DirectedKonigsbergBridge.e40_21, tail))
                    p(List.cons(step, tail))
                }
                DirectedKonigsbergBridge.e50_12 {
                    directed_incidence_count_v2_cons_e50_12(tail)
                    let a = directed_bridge_count(tail, KonigsbergBridge.e20)
                    let b = directed_bridge_count(tail, KonigsbergBridge.e40)
                    let c = directed_bridge_count(tail, KonigsbergBridge.e50)
                    let d = directed_bridge_count(tail, KonigsbergBridge.e60)
                    let e = directed_bridge_count(tail, KonigsbergBridge.e70)
                    directed_incidence_count(tail, KonigsbergVertex.v2) = a + b + c + d + e
                    Nat.1 + directed_incidence_count(tail, KonigsbergVertex.v2) = Nat.1 + (a + b + c + d + e)
                    nat_one_add_sum5(a, b, c, d, e)
                    Nat.1 + a = a + Nat.1
                    Nat.1 + a + b + c + d + e = a + Nat.1 + b + c + d + e
                    Nat.1 + b = b + Nat.1
                    a + Nat.1 + b + c + d + e = a + b + Nat.1 + c + d + e
                    a + b + Nat.1 + c + d + e = a + b + (Nat.1 + c) + d + e
                    directed_bridge_count(List.cons(DirectedKonigsbergBridge.e50_12, tail), KonigsbergBridge.e20) = directed_bridge_count(tail, KonigsbergBridge.e20)
                    directed_bridge_count(List.cons(DirectedKonigsbergBridge.e50_12, tail), KonigsbergBridge.e40) = directed_bridge_count(tail, KonigsbergBridge.e40)
                    directed_bridge_count(List.cons(DirectedKonigsbergBridge.e50_12, tail), KonigsbergBridge.e50) = Nat.1 + directed_bridge_count(tail, KonigsbergBridge.e50)
                    directed_bridge_count(List.cons(DirectedKonigsbergBridge.e50_12, tail), KonigsbergBridge.e60) = directed_bridge_count(tail, KonigsbergBridge.e60)
                    directed_bridge_count(List.cons(DirectedKonigsbergBridge.e50_12, tail), KonigsbergBridge.e70) = directed_bridge_count(tail, KonigsbergBridge.e70)
                    p(List.cons(DirectedKonigsbergBridge.e50_12, tail))
                    p(List.cons(step, tail))
                }
                DirectedKonigsbergBridge.e50_21 {
                    directed_incidence_count_v2_cons_e50_21(tail)
                    let a = directed_bridge_count(tail, KonigsbergBridge.e20)
                    let b = directed_bridge_count(tail, KonigsbergBridge.e40)
                    let c = directed_bridge_count(tail, KonigsbergBridge.e50)
                    let d = directed_bridge_count(tail, KonigsbergBridge.e60)
                    let e = directed_bridge_count(tail, KonigsbergBridge.e70)
                    directed_incidence_count(tail, KonigsbergVertex.v2) = a + b + c + d + e
                    Nat.1 + directed_incidence_count(tail, KonigsbergVertex.v2) = Nat.1 + (a + b + c + d + e)
                    nat_one_add_sum5(a, b, c, d, e)
                    Nat.1 + a = a + Nat.1
                    Nat.1 + a + b + c + d + e = a + Nat.1 + b + c + d + e
                    Nat.1 + b = b + Nat.1
                    a + Nat.1 + b + c + d + e = a + b + Nat.1 + c + d + e
                    a + b + Nat.1 + c + d + e = a + b + (Nat.1 + c) + d + e
                    directed_bridge_count(List.cons(DirectedKonigsbergBridge.e50_21, tail), KonigsbergBridge.e20) = directed_bridge_count(tail, KonigsbergBridge.e20)
                    directed_bridge_count(List.cons(DirectedKonigsbergBridge.e50_21, tail), KonigsbergBridge.e40) = directed_bridge_count(tail, KonigsbergBridge.e40)
                    directed_bridge_count(List.cons(DirectedKonigsbergBridge.e50_21, tail), KonigsbergBridge.e50) = Nat.1 + directed_bridge_count(tail, KonigsbergBridge.e50)
                    directed_bridge_count(List.cons(DirectedKonigsbergBridge.e50_21, tail), KonigsbergBridge.e60) = directed_bridge_count(tail, KonigsbergBridge.e60)
                    directed_bridge_count(List.cons(DirectedKonigsbergBridge.e50_21, tail), KonigsbergBridge.e70) = directed_bridge_count(tail, KonigsbergBridge.e70)
                    p(List.cons(DirectedKonigsbergBridge.e50_21, tail))
                    p(List.cons(step, tail))
                }
                DirectedKonigsbergBridge.e60_23 {
                    directed_incidence_count_v2_cons_e60_23(tail)
                    let a = directed_bridge_count(tail, KonigsbergBridge.e20)
                    let b = directed_bridge_count(tail, KonigsbergBridge.e40)
                    let c = directed_bridge_count(tail, KonigsbergBridge.e50)
                    let d = directed_bridge_count(tail, KonigsbergBridge.e60)
                    let e = directed_bridge_count(tail, KonigsbergBridge.e70)
                    directed_incidence_count(tail, KonigsbergVertex.v2) = a + b + c + d + e
                    Nat.1 + directed_incidence_count(tail, KonigsbergVertex.v2) = Nat.1 + (a + b + c + d + e)
                    nat_one_add_sum5(a, b, c, d, e)
                    Nat.1 + a = a + Nat.1
                    Nat.1 + a + b + c + d + e = a + Nat.1 + b + c + d + e
                    Nat.1 + b = b + Nat.1
                    a + Nat.1 + b + c + d + e = a + b + Nat.1 + c + d + e
                    Nat.1 + c = c + Nat.1
                    a + b + Nat.1 + c + d + e = a + b + c + Nat.1 + d + e
                    a + b + c + Nat.1 + d + e = a + b + c + (Nat.1 + d) + e
                    directed_bridge_count(List.cons(DirectedKonigsbergBridge.e60_23, tail), KonigsbergBridge.e20) = directed_bridge_count(tail, KonigsbergBridge.e20)
                    directed_bridge_count(List.cons(DirectedKonigsbergBridge.e60_23, tail), KonigsbergBridge.e40) = directed_bridge_count(tail, KonigsbergBridge.e40)
                    directed_bridge_count(List.cons(DirectedKonigsbergBridge.e60_23, tail), KonigsbergBridge.e50) = directed_bridge_count(tail, KonigsbergBridge.e50)
                    directed_bridge_count(List.cons(DirectedKonigsbergBridge.e60_23, tail), KonigsbergBridge.e60) = Nat.1 + directed_bridge_count(tail, KonigsbergBridge.e60)
                    directed_bridge_count(List.cons(DirectedKonigsbergBridge.e60_23, tail), KonigsbergBridge.e70) = directed_bridge_count(tail, KonigsbergBridge.e70)
                    p(List.cons(DirectedKonigsbergBridge.e60_23, tail))
                    p(List.cons(step, tail))
                }
                DirectedKonigsbergBridge.e60_32 {
                    directed_incidence_count_v2_cons_e60_32(tail)
                    let a = directed_bridge_count(tail, KonigsbergBridge.e20)
                    let b = directed_bridge_count(tail, KonigsbergBridge.e40)
                    let c = directed_bridge_count(tail, KonigsbergBridge.e50)
                    let d = directed_bridge_count(tail, KonigsbergBridge.e60)
                    let e = directed_bridge_count(tail, KonigsbergBridge.e70)
                    directed_incidence_count(tail, KonigsbergVertex.v2) = a + b + c + d + e
                    Nat.1 + directed_incidence_count(tail, KonigsbergVertex.v2) = Nat.1 + (a + b + c + d + e)
                    nat_one_add_sum5(a, b, c, d, e)
                    Nat.1 + a = a + Nat.1
                    Nat.1 + a + b + c + d + e = a + Nat.1 + b + c + d + e
                    Nat.1 + b = b + Nat.1
                    a + Nat.1 + b + c + d + e = a + b + Nat.1 + c + d + e
                    Nat.1 + c = c + Nat.1
                    a + b + Nat.1 + c + d + e = a + b + c + Nat.1 + d + e
                    a + b + c + Nat.1 + d + e = a + b + c + (Nat.1 + d) + e
                    directed_bridge_count(List.cons(DirectedKonigsbergBridge.e60_32, tail), KonigsbergBridge.e20) = directed_bridge_count(tail, KonigsbergBridge.e20)
                    directed_bridge_count(List.cons(DirectedKonigsbergBridge.e60_32, tail), KonigsbergBridge.e40) = directed_bridge_count(tail, KonigsbergBridge.e40)
                    directed_bridge_count(List.cons(DirectedKonigsbergBridge.e60_32, tail), KonigsbergBridge.e50) = directed_bridge_count(tail, KonigsbergBridge.e50)
                    directed_bridge_count(List.cons(DirectedKonigsbergBridge.e60_32, tail), KonigsbergBridge.e60) = Nat.1 + directed_bridge_count(tail, KonigsbergBridge.e60)
                    directed_bridge_count(List.cons(DirectedKonigsbergBridge.e60_32, tail), KonigsbergBridge.e70) = directed_bridge_count(tail, KonigsbergBridge.e70)
                    p(List.cons(DirectedKonigsbergBridge.e60_32, tail))
                    p(List.cons(step, tail))
                }
                DirectedKonigsbergBridge.e70_23 {
                    directed_incidence_count_v2_cons_e70_23(tail)
                    let a = directed_bridge_count(tail, KonigsbergBridge.e20)
                    let b = directed_bridge_count(tail, KonigsbergBridge.e40)
                    let c = directed_bridge_count(tail, KonigsbergBridge.e50)
                    let d = directed_bridge_count(tail, KonigsbergBridge.e60)
                    let e = directed_bridge_count(tail, KonigsbergBridge.e70)
                    directed_incidence_count(tail, KonigsbergVertex.v2) = a + b + c + d + e
                    Nat.1 + directed_incidence_count(tail, KonigsbergVertex.v2) = Nat.1 + (a + b + c + d + e)
                    nat_one_add_sum5(a, b, c, d, e)
                    Nat.1 + a = a + Nat.1
                    Nat.1 + a + b + c + d + e = a + Nat.1 + b + c + d + e
                    Nat.1 + b = b + Nat.1
                    a + Nat.1 + b + c + d + e = a + b + Nat.1 + c + d + e
                    Nat.1 + c = c + Nat.1
                    a + b + Nat.1 + c + d + e = a + b + c + Nat.1 + d + e
                    Nat.1 + d = d + Nat.1
                    a + b + c + Nat.1 + d + e = a + b + c + d + Nat.1 + e
                    a + b + c + d + Nat.1 + e = a + b + c + d + (Nat.1 + e)
                    directed_bridge_count(List.cons(DirectedKonigsbergBridge.e70_23, tail), KonigsbergBridge.e20) = directed_bridge_count(tail, KonigsbergBridge.e20)
                    directed_bridge_count(List.cons(DirectedKonigsbergBridge.e70_23, tail), KonigsbergBridge.e40) = directed_bridge_count(tail, KonigsbergBridge.e40)
                    directed_bridge_count(List.cons(DirectedKonigsbergBridge.e70_23, tail), KonigsbergBridge.e50) = directed_bridge_count(tail, KonigsbergBridge.e50)
                    directed_bridge_count(List.cons(DirectedKonigsbergBridge.e70_23, tail), KonigsbergBridge.e60) = directed_bridge_count(tail, KonigsbergBridge.e60)
                    directed_bridge_count(List.cons(DirectedKonigsbergBridge.e70_23, tail), KonigsbergBridge.e70) = Nat.1 + directed_bridge_count(tail, KonigsbergBridge.e70)
                    p(List.cons(DirectedKonigsbergBridge.e70_23, tail))
                    p(List.cons(step, tail))
                }
                DirectedKonigsbergBridge.e70_32 {
                    directed_incidence_count_v2_cons_e70_32(tail)
                    let a = directed_bridge_count(tail, KonigsbergBridge.e20)
                    let b = directed_bridge_count(tail, KonigsbergBridge.e40)
                    let c = directed_bridge_count(tail, KonigsbergBridge.e50)
                    let d = directed_bridge_count(tail, KonigsbergBridge.e60)
                    let e = directed_bridge_count(tail, KonigsbergBridge.e70)
                    directed_incidence_count(tail, KonigsbergVertex.v2) = a + b + c + d + e
                    Nat.1 + directed_incidence_count(tail, KonigsbergVertex.v2) = Nat.1 + (a + b + c + d + e)
                    nat_one_add_sum5(a, b, c, d, e)
                    Nat.1 + a = a + Nat.1
                    Nat.1 + a + b + c + d + e = a + Nat.1 + b + c + d + e
                    Nat.1 + b = b + Nat.1
                    a + Nat.1 + b + c + d + e = a + b + Nat.1 + c + d + e
                    Nat.1 + c = c + Nat.1
                    a + b + Nat.1 + c + d + e = a + b + c + Nat.1 + d + e
                    Nat.1 + d = d + Nat.1
                    a + b + c + Nat.1 + d + e = a + b + c + d + Nat.1 + e
                    a + b + c + d + Nat.1 + e = a + b + c + d + (Nat.1 + e)
                    directed_bridge_count(List.cons(DirectedKonigsbergBridge.e70_32, tail), KonigsbergBridge.e20) = directed_bridge_count(tail, KonigsbergBridge.e20)
                    directed_bridge_count(List.cons(DirectedKonigsbergBridge.e70_32, tail), KonigsbergBridge.e40) = directed_bridge_count(tail, KonigsbergBridge.e40)
                    directed_bridge_count(List.cons(DirectedKonigsbergBridge.e70_32, tail), KonigsbergBridge.e50) = directed_bridge_count(tail, KonigsbergBridge.e50)
                    directed_bridge_count(List.cons(DirectedKonigsbergBridge.e70_32, tail), KonigsbergBridge.e60) = directed_bridge_count(tail, KonigsbergBridge.e60)
                    directed_bridge_count(List.cons(DirectedKonigsbergBridge.e70_32, tail), KonigsbergBridge.e70) = Nat.1 + directed_bridge_count(tail, KonigsbergBridge.e70)
                    p(List.cons(DirectedKonigsbergBridge.e70_32, tail))
                    p(List.cons(step, tail))
                }
            }
        }
    }

    p(steps)
}

/// A crossing of bridge `e10` does not touch `v3`.
theorem directed_incidence_count_v3_cons_e10_01(tail: List[DirectedKonigsbergBridge]) {
    directed_incidence_count(List.cons(DirectedKonigsbergBridge.e10_01, tail), KonigsbergVertex.v3) =
        directed_incidence_count(tail, KonigsbergVertex.v3)
} by {
    directed_incidence_count_cons(DirectedKonigsbergBridge.e10_01, tail, KonigsbergVertex.v3)
    vertex_indicator(directed_source(DirectedKonigsbergBridge.e10_01), KonigsbergVertex.v3) = Nat.0
    vertex_indicator(directed_target(DirectedKonigsbergBridge.e10_01), KonigsbergVertex.v3) = Nat.0
}

/// A crossing of bridge `e10` does not touch `v3`.
theorem directed_incidence_count_v3_cons_e10_10(tail: List[DirectedKonigsbergBridge]) {
    directed_incidence_count(List.cons(DirectedKonigsbergBridge.e10_10, tail), KonigsbergVertex.v3) =
        directed_incidence_count(tail, KonigsbergVertex.v3)
} by {
    directed_incidence_count_cons(DirectedKonigsbergBridge.e10_10, tail, KonigsbergVertex.v3)
    vertex_indicator(directed_source(DirectedKonigsbergBridge.e10_10), KonigsbergVertex.v3) = Nat.0
    vertex_indicator(directed_target(DirectedKonigsbergBridge.e10_10), KonigsbergVertex.v3) = Nat.0
}

/// A crossing of bridge `e20` does not touch `v3`.
theorem directed_incidence_count_v3_cons_e20_02(tail: List[DirectedKonigsbergBridge]) {
    directed_incidence_count(List.cons(DirectedKonigsbergBridge.e20_02, tail), KonigsbergVertex.v3) =
        directed_incidence_count(tail, KonigsbergVertex.v3)
} by {
    directed_incidence_count_cons(DirectedKonigsbergBridge.e20_02, tail, KonigsbergVertex.v3)
    vertex_indicator(directed_source(DirectedKonigsbergBridge.e20_02), KonigsbergVertex.v3) = Nat.0
    vertex_indicator(directed_target(DirectedKonigsbergBridge.e20_02), KonigsbergVertex.v3) = Nat.0
}

/// A crossing of bridge `e20` does not touch `v3`.
theorem directed_incidence_count_v3_cons_e20_20(tail: List[DirectedKonigsbergBridge]) {
    directed_incidence_count(List.cons(DirectedKonigsbergBridge.e20_20, tail), KonigsbergVertex.v3) =
        directed_incidence_count(tail, KonigsbergVertex.v3)
} by {
    directed_incidence_count_cons(DirectedKonigsbergBridge.e20_20, tail, KonigsbergVertex.v3)
    vertex_indicator(directed_source(DirectedKonigsbergBridge.e20_20), KonigsbergVertex.v3) = Nat.0
    vertex_indicator(directed_target(DirectedKonigsbergBridge.e20_20), KonigsbergVertex.v3) = Nat.0
}

/// A crossing of bridge `e30` from `v0` to `v3` adds one incident crossing at `v3`.
theorem directed_incidence_count_v3_cons_e30_03(tail: List[DirectedKonigsbergBridge]) {
    directed_incidence_count(List.cons(DirectedKonigsbergBridge.e30_03, tail), KonigsbergVertex.v3) =
        Nat.1 + directed_incidence_count(tail, KonigsbergVertex.v3)
} by {
    directed_incidence_count_cons(DirectedKonigsbergBridge.e30_03, tail, KonigsbergVertex.v3)
    vertex_indicator(directed_source(DirectedKonigsbergBridge.e30_03), KonigsbergVertex.v3) = Nat.0
    vertex_indicator(directed_target(DirectedKonigsbergBridge.e30_03), KonigsbergVertex.v3) = Nat.1
}

/// A crossing of bridge `e30` from `v3` to `v0` adds one incident crossing at `v3`.
theorem directed_incidence_count_v3_cons_e30_30(tail: List[DirectedKonigsbergBridge]) {
    directed_incidence_count(List.cons(DirectedKonigsbergBridge.e30_30, tail), KonigsbergVertex.v3) =
        Nat.1 + directed_incidence_count(tail, KonigsbergVertex.v3)
} by {
    directed_incidence_count_cons(DirectedKonigsbergBridge.e30_30, tail, KonigsbergVertex.v3)
    vertex_indicator(directed_source(DirectedKonigsbergBridge.e30_30), KonigsbergVertex.v3) = Nat.1
    vertex_indicator(directed_target(DirectedKonigsbergBridge.e30_30), KonigsbergVertex.v3) = Nat.0
}

/// A crossing of bridge `e60` from `v2` to `v3` adds one incident crossing at `v3`.
theorem directed_incidence_count_v3_cons_e60_23(tail: List[DirectedKonigsbergBridge]) {
    directed_incidence_count(List.cons(DirectedKonigsbergBridge.e60_23, tail), KonigsbergVertex.v3) =
        Nat.1 + directed_incidence_count(tail, KonigsbergVertex.v3)
} by {
    directed_incidence_count_cons(DirectedKonigsbergBridge.e60_23, tail, KonigsbergVertex.v3)
    vertex_indicator(directed_source(DirectedKonigsbergBridge.e60_23), KonigsbergVertex.v3) = Nat.0
    vertex_indicator(directed_target(DirectedKonigsbergBridge.e60_23), KonigsbergVertex.v3) = Nat.1
}

/// A crossing of bridge `e60` from `v3` to `v2` adds one incident crossing at `v3`.
theorem directed_incidence_count_v3_cons_e60_32(tail: List[DirectedKonigsbergBridge]) {
    directed_incidence_count(List.cons(DirectedKonigsbergBridge.e60_32, tail), KonigsbergVertex.v3) =
        Nat.1 + directed_incidence_count(tail, KonigsbergVertex.v3)
} by {
    directed_incidence_count_cons(DirectedKonigsbergBridge.e60_32, tail, KonigsbergVertex.v3)
    vertex_indicator(directed_source(DirectedKonigsbergBridge.e60_32), KonigsbergVertex.v3) = Nat.1
    vertex_indicator(directed_target(DirectedKonigsbergBridge.e60_32), KonigsbergVertex.v3) = Nat.0
}

/// A crossing of bridge `e70` from `v2` to `v3` adds one incident crossing at `v3`.
theorem directed_incidence_count_v3_cons_e70_23(tail: List[DirectedKonigsbergBridge]) {
    directed_incidence_count(List.cons(DirectedKonigsbergBridge.e70_23, tail), KonigsbergVertex.v3) =
        Nat.1 + directed_incidence_count(tail, KonigsbergVertex.v3)
} by {
    directed_incidence_count_cons(DirectedKonigsbergBridge.e70_23, tail, KonigsbergVertex.v3)
    vertex_indicator(directed_source(DirectedKonigsbergBridge.e70_23), KonigsbergVertex.v3) = Nat.0
    vertex_indicator(directed_target(DirectedKonigsbergBridge.e70_23), KonigsbergVertex.v3) = Nat.1
}

/// A crossing of bridge `e70` from `v3` to `v2` adds one incident crossing at `v3`.
theorem directed_incidence_count_v3_cons_e70_32(tail: List[DirectedKonigsbergBridge]) {
    directed_incidence_count(List.cons(DirectedKonigsbergBridge.e70_32, tail), KonigsbergVertex.v3) =
        Nat.1 + directed_incidence_count(tail, KonigsbergVertex.v3)
} by {
    directed_incidence_count_cons(DirectedKonigsbergBridge.e70_32, tail, KonigsbergVertex.v3)
    vertex_indicator(directed_source(DirectedKonigsbergBridge.e70_32), KonigsbergVertex.v3) = Nat.1
    vertex_indicator(directed_target(DirectedKonigsbergBridge.e70_32), KonigsbergVertex.v3) = Nat.0
}

/// A crossing of bridge `e40` does not touch `v3`.
theorem directed_incidence_count_v3_cons_e40_12(tail: List[DirectedKonigsbergBridge]) {
    directed_incidence_count(List.cons(DirectedKonigsbergBridge.e40_12, tail), KonigsbergVertex.v3) =
        directed_incidence_count(tail, KonigsbergVertex.v3)
} by {
    directed_incidence_count_cons(DirectedKonigsbergBridge.e40_12, tail, KonigsbergVertex.v3)
    vertex_indicator(directed_source(DirectedKonigsbergBridge.e40_12), KonigsbergVertex.v3) = Nat.0
    vertex_indicator(directed_target(DirectedKonigsbergBridge.e40_12), KonigsbergVertex.v3) = Nat.0
}

/// A crossing of bridge `e40` does not touch `v3`.
theorem directed_incidence_count_v3_cons_e40_21(tail: List[DirectedKonigsbergBridge]) {
    directed_incidence_count(List.cons(DirectedKonigsbergBridge.e40_21, tail), KonigsbergVertex.v3) =
        directed_incidence_count(tail, KonigsbergVertex.v3)
} by {
    directed_incidence_count_cons(DirectedKonigsbergBridge.e40_21, tail, KonigsbergVertex.v3)
    vertex_indicator(directed_source(DirectedKonigsbergBridge.e40_21), KonigsbergVertex.v3) = Nat.0
    vertex_indicator(directed_target(DirectedKonigsbergBridge.e40_21), KonigsbergVertex.v3) = Nat.0
}

/// A crossing of bridge `e50` does not touch `v3`.
theorem directed_incidence_count_v3_cons_e50_12(tail: List[DirectedKonigsbergBridge]) {
    directed_incidence_count(List.cons(DirectedKonigsbergBridge.e50_12, tail), KonigsbergVertex.v3) =
        directed_incidence_count(tail, KonigsbergVertex.v3)
} by {
    directed_incidence_count_cons(DirectedKonigsbergBridge.e50_12, tail, KonigsbergVertex.v3)
    vertex_indicator(directed_source(DirectedKonigsbergBridge.e50_12), KonigsbergVertex.v3) = Nat.0
    vertex_indicator(directed_target(DirectedKonigsbergBridge.e50_12), KonigsbergVertex.v3) = Nat.0
}

/// A crossing of bridge `e50` does not touch `v3`.
theorem directed_incidence_count_v3_cons_e50_21(tail: List[DirectedKonigsbergBridge]) {
    directed_incidence_count(List.cons(DirectedKonigsbergBridge.e50_21, tail), KonigsbergVertex.v3) =
        directed_incidence_count(tail, KonigsbergVertex.v3)
} by {
    directed_incidence_count_cons(DirectedKonigsbergBridge.e50_21, tail, KonigsbergVertex.v3)
    vertex_indicator(directed_source(DirectedKonigsbergBridge.e50_21), KonigsbergVertex.v3) = Nat.0
    vertex_indicator(directed_target(DirectedKonigsbergBridge.e50_21), KonigsbergVertex.v3) = Nat.0
}

/// The incident crossings at `v3` are exactly the uses of bridges `e30`, `e60`, and `e70`.
theorem directed_incidence_count_v3(steps: List[DirectedKonigsbergBridge]) {
    directed_incidence_count(steps, KonigsbergVertex.v3) =
        directed_bridge_count(steps, KonigsbergBridge.e30) +
        directed_bridge_count(steps, KonigsbergBridge.e60) +
        directed_bridge_count(steps, KonigsbergBridge.e70)
} by {
    define p(xs: List[DirectedKonigsbergBridge]) -> Bool {
        directed_incidence_count(xs, KonigsbergVertex.v3) =
            directed_bridge_count(xs, KonigsbergBridge.e30) +
            directed_bridge_count(xs, KonigsbergBridge.e60) +
            directed_bridge_count(xs, KonigsbergBridge.e70)
    }

    directed_incidence_count(List.nil[DirectedKonigsbergBridge], KonigsbergVertex.v3) = Nat.0
    directed_bridge_count(List.nil[DirectedKonigsbergBridge], KonigsbergBridge.e30) = Nat.0
    directed_bridge_count(List.nil[DirectedKonigsbergBridge], KonigsbergBridge.e60) = Nat.0
    directed_bridge_count(List.nil[DirectedKonigsbergBridge], KonigsbergBridge.e70) = Nat.0
    Nat.0 = Nat.0 + Nat.0 + Nat.0
    p(List.nil[DirectedKonigsbergBridge])

    forall(step: DirectedKonigsbergBridge, tail: List[DirectedKonigsbergBridge]) {
        if p(tail) {
            match step {
                DirectedKonigsbergBridge.e10_01 {
                    directed_incidence_count_v3_cons_e10_01(tail)
                    directed_bridge_count(List.cons(DirectedKonigsbergBridge.e10_01, tail), KonigsbergBridge.e30) = directed_bridge_count(tail, KonigsbergBridge.e30)
                    directed_bridge_count(List.cons(DirectedKonigsbergBridge.e10_01, tail), KonigsbergBridge.e60) = directed_bridge_count(tail, KonigsbergBridge.e60)
                    directed_bridge_count(List.cons(DirectedKonigsbergBridge.e10_01, tail), KonigsbergBridge.e70) = directed_bridge_count(tail, KonigsbergBridge.e70)
                    p(List.cons(DirectedKonigsbergBridge.e10_01, tail))
                    p(List.cons(step, tail))
                }
                DirectedKonigsbergBridge.e10_10 {
                    directed_incidence_count_v3_cons_e10_10(tail)
                    directed_bridge_count(List.cons(DirectedKonigsbergBridge.e10_10, tail), KonigsbergBridge.e30) = directed_bridge_count(tail, KonigsbergBridge.e30)
                    directed_bridge_count(List.cons(DirectedKonigsbergBridge.e10_10, tail), KonigsbergBridge.e60) = directed_bridge_count(tail, KonigsbergBridge.e60)
                    directed_bridge_count(List.cons(DirectedKonigsbergBridge.e10_10, tail), KonigsbergBridge.e70) = directed_bridge_count(tail, KonigsbergBridge.e70)
                    p(List.cons(DirectedKonigsbergBridge.e10_10, tail))
                    p(List.cons(step, tail))
                }
                DirectedKonigsbergBridge.e20_02 {
                    directed_incidence_count_v3_cons_e20_02(tail)
                    directed_bridge_count(List.cons(DirectedKonigsbergBridge.e20_02, tail), KonigsbergBridge.e30) = directed_bridge_count(tail, KonigsbergBridge.e30)
                    directed_bridge_count(List.cons(DirectedKonigsbergBridge.e20_02, tail), KonigsbergBridge.e60) = directed_bridge_count(tail, KonigsbergBridge.e60)
                    directed_bridge_count(List.cons(DirectedKonigsbergBridge.e20_02, tail), KonigsbergBridge.e70) = directed_bridge_count(tail, KonigsbergBridge.e70)
                    p(List.cons(DirectedKonigsbergBridge.e20_02, tail))
                    p(List.cons(step, tail))
                }
                DirectedKonigsbergBridge.e20_20 {
                    directed_incidence_count_v3_cons_e20_20(tail)
                    directed_bridge_count(List.cons(DirectedKonigsbergBridge.e20_20, tail), KonigsbergBridge.e30) = directed_bridge_count(tail, KonigsbergBridge.e30)
                    directed_bridge_count(List.cons(DirectedKonigsbergBridge.e20_20, tail), KonigsbergBridge.e60) = directed_bridge_count(tail, KonigsbergBridge.e60)
                    directed_bridge_count(List.cons(DirectedKonigsbergBridge.e20_20, tail), KonigsbergBridge.e70) = directed_bridge_count(tail, KonigsbergBridge.e70)
                    p(List.cons(DirectedKonigsbergBridge.e20_20, tail))
                    p(List.cons(step, tail))
                }
                DirectedKonigsbergBridge.e30_03 {
                    directed_incidence_count_v3_cons_e30_03(tail)
                    let a = directed_bridge_count(tail, KonigsbergBridge.e30)
                    let b = directed_bridge_count(tail, KonigsbergBridge.e60)
                    let c = directed_bridge_count(tail, KonigsbergBridge.e70)
                    directed_incidence_count(tail, KonigsbergVertex.v3) = a + b + c
                    Nat.1 + directed_incidence_count(tail, KonigsbergVertex.v3) = Nat.1 + (a + b + c)
                    Nat.1 + (a + b + c) = Nat.1 + a + b + c
                    directed_bridge_count(List.cons(DirectedKonigsbergBridge.e30_03, tail), KonigsbergBridge.e30) = Nat.1 + directed_bridge_count(tail, KonigsbergBridge.e30)
                    directed_bridge_count(List.cons(DirectedKonigsbergBridge.e30_03, tail), KonigsbergBridge.e60) = directed_bridge_count(tail, KonigsbergBridge.e60)
                    directed_bridge_count(List.cons(DirectedKonigsbergBridge.e30_03, tail), KonigsbergBridge.e70) = directed_bridge_count(tail, KonigsbergBridge.e70)
                    p(List.cons(DirectedKonigsbergBridge.e30_03, tail))
                    p(List.cons(step, tail))
                }
                DirectedKonigsbergBridge.e30_30 {
                    directed_incidence_count_v3_cons_e30_30(tail)
                    let a = directed_bridge_count(tail, KonigsbergBridge.e30)
                    let b = directed_bridge_count(tail, KonigsbergBridge.e60)
                    let c = directed_bridge_count(tail, KonigsbergBridge.e70)
                    directed_incidence_count(tail, KonigsbergVertex.v3) = a + b + c
                    Nat.1 + directed_incidence_count(tail, KonigsbergVertex.v3) = Nat.1 + (a + b + c)
                    Nat.1 + (a + b + c) = Nat.1 + a + b + c
                    directed_bridge_count(List.cons(DirectedKonigsbergBridge.e30_30, tail), KonigsbergBridge.e30) = Nat.1 + directed_bridge_count(tail, KonigsbergBridge.e30)
                    directed_bridge_count(List.cons(DirectedKonigsbergBridge.e30_30, tail), KonigsbergBridge.e60) = directed_bridge_count(tail, KonigsbergBridge.e60)
                    directed_bridge_count(List.cons(DirectedKonigsbergBridge.e30_30, tail), KonigsbergBridge.e70) = directed_bridge_count(tail, KonigsbergBridge.e70)
                    p(List.cons(DirectedKonigsbergBridge.e30_30, tail))
                    p(List.cons(step, tail))
                }
                DirectedKonigsbergBridge.e40_12 {
                    directed_incidence_count_v3_cons_e40_12(tail)
                    directed_bridge_count(List.cons(DirectedKonigsbergBridge.e40_12, tail), KonigsbergBridge.e30) = directed_bridge_count(tail, KonigsbergBridge.e30)
                    directed_bridge_count(List.cons(DirectedKonigsbergBridge.e40_12, tail), KonigsbergBridge.e60) = directed_bridge_count(tail, KonigsbergBridge.e60)
                    directed_bridge_count(List.cons(DirectedKonigsbergBridge.e40_12, tail), KonigsbergBridge.e70) = directed_bridge_count(tail, KonigsbergBridge.e70)
                    p(List.cons(DirectedKonigsbergBridge.e40_12, tail))
                    p(List.cons(step, tail))
                }
                DirectedKonigsbergBridge.e40_21 {
                    directed_incidence_count_v3_cons_e40_21(tail)
                    directed_bridge_count(List.cons(DirectedKonigsbergBridge.e40_21, tail), KonigsbergBridge.e30) = directed_bridge_count(tail, KonigsbergBridge.e30)
                    directed_bridge_count(List.cons(DirectedKonigsbergBridge.e40_21, tail), KonigsbergBridge.e60) = directed_bridge_count(tail, KonigsbergBridge.e60)
                    directed_bridge_count(List.cons(DirectedKonigsbergBridge.e40_21, tail), KonigsbergBridge.e70) = directed_bridge_count(tail, KonigsbergBridge.e70)
                    p(List.cons(DirectedKonigsbergBridge.e40_21, tail))
                    p(List.cons(step, tail))
                }
                DirectedKonigsbergBridge.e50_12 {
                    directed_incidence_count_v3_cons_e50_12(tail)
                    directed_bridge_count(List.cons(DirectedKonigsbergBridge.e50_12, tail), KonigsbergBridge.e30) = directed_bridge_count(tail, KonigsbergBridge.e30)
                    directed_bridge_count(List.cons(DirectedKonigsbergBridge.e50_12, tail), KonigsbergBridge.e60) = directed_bridge_count(tail, KonigsbergBridge.e60)
                    directed_bridge_count(List.cons(DirectedKonigsbergBridge.e50_12, tail), KonigsbergBridge.e70) = directed_bridge_count(tail, KonigsbergBridge.e70)
                    p(List.cons(DirectedKonigsbergBridge.e50_12, tail))
                    p(List.cons(step, tail))
                }
                DirectedKonigsbergBridge.e50_21 {
                    directed_incidence_count_v3_cons_e50_21(tail)
                    directed_bridge_count(List.cons(DirectedKonigsbergBridge.e50_21, tail), KonigsbergBridge.e30) = directed_bridge_count(tail, KonigsbergBridge.e30)
                    directed_bridge_count(List.cons(DirectedKonigsbergBridge.e50_21, tail), KonigsbergBridge.e60) = directed_bridge_count(tail, KonigsbergBridge.e60)
                    directed_bridge_count(List.cons(DirectedKonigsbergBridge.e50_21, tail), KonigsbergBridge.e70) = directed_bridge_count(tail, KonigsbergBridge.e70)
                    p(List.cons(DirectedKonigsbergBridge.e50_21, tail))
                    p(List.cons(step, tail))
                }
                DirectedKonigsbergBridge.e60_23 {
                    directed_incidence_count_v3_cons_e60_23(tail)
                    let a = directed_bridge_count(tail, KonigsbergBridge.e30)
                    let b = directed_bridge_count(tail, KonigsbergBridge.e60)
                    let c = directed_bridge_count(tail, KonigsbergBridge.e70)
                    directed_incidence_count(tail, KonigsbergVertex.v3) = a + b + c
                    Nat.1 + directed_incidence_count(tail, KonigsbergVertex.v3) = Nat.1 + (a + b + c)
                    Nat.1 + (a + b + c) = Nat.1 + a + b + c
                    Nat.1 + a = a + Nat.1
                    Nat.1 + a + b + c = a + Nat.1 + b + c
                    a + Nat.1 + b + c = a + (Nat.1 + b) + c
                    directed_bridge_count(List.cons(DirectedKonigsbergBridge.e60_23, tail), KonigsbergBridge.e30) = directed_bridge_count(tail, KonigsbergBridge.e30)
                    directed_bridge_count(List.cons(DirectedKonigsbergBridge.e60_23, tail), KonigsbergBridge.e60) = Nat.1 + directed_bridge_count(tail, KonigsbergBridge.e60)
                    directed_bridge_count(List.cons(DirectedKonigsbergBridge.e60_23, tail), KonigsbergBridge.e70) = directed_bridge_count(tail, KonigsbergBridge.e70)
                    p(List.cons(DirectedKonigsbergBridge.e60_23, tail))
                    p(List.cons(step, tail))
                }
                DirectedKonigsbergBridge.e60_32 {
                    directed_incidence_count_v3_cons_e60_32(tail)
                    let a = directed_bridge_count(tail, KonigsbergBridge.e30)
                    let b = directed_bridge_count(tail, KonigsbergBridge.e60)
                    let c = directed_bridge_count(tail, KonigsbergBridge.e70)
                    directed_incidence_count(tail, KonigsbergVertex.v3) = a + b + c
                    Nat.1 + directed_incidence_count(tail, KonigsbergVertex.v3) = Nat.1 + (a + b + c)
                    Nat.1 + (a + b + c) = Nat.1 + a + b + c
                    Nat.1 + a = a + Nat.1
                    Nat.1 + a + b + c = a + Nat.1 + b + c
                    a + Nat.1 + b + c = a + (Nat.1 + b) + c
                    directed_bridge_count(List.cons(DirectedKonigsbergBridge.e60_32, tail), KonigsbergBridge.e30) = directed_bridge_count(tail, KonigsbergBridge.e30)
                    directed_bridge_count(List.cons(DirectedKonigsbergBridge.e60_32, tail), KonigsbergBridge.e60) = Nat.1 + directed_bridge_count(tail, KonigsbergBridge.e60)
                    directed_bridge_count(List.cons(DirectedKonigsbergBridge.e60_32, tail), KonigsbergBridge.e70) = directed_bridge_count(tail, KonigsbergBridge.e70)
                    p(List.cons(DirectedKonigsbergBridge.e60_32, tail))
                    p(List.cons(step, tail))
                }
                DirectedKonigsbergBridge.e70_23 {
                    directed_incidence_count_v3_cons_e70_23(tail)
                    let a = directed_bridge_count(tail, KonigsbergBridge.e30)
                    let b = directed_bridge_count(tail, KonigsbergBridge.e60)
                    let c = directed_bridge_count(tail, KonigsbergBridge.e70)
                    directed_incidence_count(tail, KonigsbergVertex.v3) = a + b + c
                    Nat.1 + directed_incidence_count(tail, KonigsbergVertex.v3) = Nat.1 + (a + b + c)
                    Nat.1 + (a + b + c) = Nat.1 + a + b + c
                    Nat.1 + a = a + Nat.1
                    Nat.1 + a + b + c = a + Nat.1 + b + c
                    Nat.1 + b = b + Nat.1
                    a + Nat.1 + b + c = a + b + Nat.1 + c
                    a + b + Nat.1 + c = a + b + (Nat.1 + c)
                    directed_bridge_count(List.cons(DirectedKonigsbergBridge.e70_23, tail), KonigsbergBridge.e30) = directed_bridge_count(tail, KonigsbergBridge.e30)
                    directed_bridge_count(List.cons(DirectedKonigsbergBridge.e70_23, tail), KonigsbergBridge.e60) = directed_bridge_count(tail, KonigsbergBridge.e60)
                    directed_bridge_count(List.cons(DirectedKonigsbergBridge.e70_23, tail), KonigsbergBridge.e70) = Nat.1 + directed_bridge_count(tail, KonigsbergBridge.e70)
                    p(List.cons(DirectedKonigsbergBridge.e70_23, tail))
                    p(List.cons(step, tail))
                }
                DirectedKonigsbergBridge.e70_32 {
                    directed_incidence_count_v3_cons_e70_32(tail)
                    let a = directed_bridge_count(tail, KonigsbergBridge.e30)
                    let b = directed_bridge_count(tail, KonigsbergBridge.e60)
                    let c = directed_bridge_count(tail, KonigsbergBridge.e70)
                    directed_incidence_count(tail, KonigsbergVertex.v3) = a + b + c
                    Nat.1 + directed_incidence_count(tail, KonigsbergVertex.v3) = Nat.1 + (a + b + c)
                    Nat.1 + (a + b + c) = Nat.1 + a + b + c
                    Nat.1 + a = a + Nat.1
                    Nat.1 + a + b + c = a + Nat.1 + b + c
                    Nat.1 + b = b + Nat.1
                    a + Nat.1 + b + c = a + b + Nat.1 + c
                    a + b + Nat.1 + c = a + b + (Nat.1 + c)
                    directed_bridge_count(List.cons(DirectedKonigsbergBridge.e70_32, tail), KonigsbergBridge.e30) = directed_bridge_count(tail, KonigsbergBridge.e30)
                    directed_bridge_count(List.cons(DirectedKonigsbergBridge.e70_32, tail), KonigsbergBridge.e60) = directed_bridge_count(tail, KonigsbergBridge.e60)
                    directed_bridge_count(List.cons(DirectedKonigsbergBridge.e70_32, tail), KonigsbergBridge.e70) = Nat.1 + directed_bridge_count(tail, KonigsbergBridge.e70)
                    p(List.cons(DirectedKonigsbergBridge.e70_32, tail))
                    p(List.cons(step, tail))
                }
            }
        }
    }

    p(steps)
}

/// Using every directed bridge once gives the fixed directed degree counts.
theorem directed_uses_every_bridge_once_imp_degree_counts(steps: List[DirectedKonigsbergBridge]) {
    directed_uses_every_bridge_once(steps) implies konigsberg_directed_degree_counts(steps)
} by {
    if directed_uses_every_bridge_once(steps) {
        directed_uses_every_bridge_once(steps) =
            (directed_bridge_count(steps, KonigsbergBridge.e10) = Nat.1 and
            directed_bridge_count(steps, KonigsbergBridge.e20) = Nat.1 and
            directed_bridge_count(steps, KonigsbergBridge.e30) = Nat.1 and
            directed_bridge_count(steps, KonigsbergBridge.e40) = Nat.1 and
            directed_bridge_count(steps, KonigsbergBridge.e50) = Nat.1 and
            directed_bridge_count(steps, KonigsbergBridge.e60) = Nat.1 and
            directed_bridge_count(steps, KonigsbergBridge.e70) = Nat.1)
        directed_bridge_count(steps, KonigsbergBridge.e10) = Nat.1
        directed_bridge_count(steps, KonigsbergBridge.e20) = Nat.1
        directed_bridge_count(steps, KonigsbergBridge.e30) = Nat.1
        directed_bridge_count(steps, KonigsbergBridge.e40) = Nat.1
        directed_bridge_count(steps, KonigsbergBridge.e50) = Nat.1
        directed_bridge_count(steps, KonigsbergBridge.e60) = Nat.1
        directed_bridge_count(steps, KonigsbergBridge.e70) = Nat.1

        directed_incidence_count_v0(steps)
        directed_incidence_count_v1(steps)
        directed_incidence_count_v2(steps)
        directed_incidence_count_v3(steps)
        directed_incidence_count(steps, KonigsbergVertex.v0) = Nat.1 + Nat.1 + Nat.1
        Nat.1 + Nat.1 + Nat.1 = Nat.3
        directed_incidence_count(steps, KonigsbergVertex.v0) = Nat.3
        directed_incidence_count(steps, KonigsbergVertex.v1) = Nat.1 + Nat.1 + Nat.1
        directed_incidence_count(steps, KonigsbergVertex.v1) = Nat.3
        directed_incidence_count(steps, KonigsbergVertex.v2) = Nat.1 + Nat.1 + Nat.1 + Nat.1 + Nat.1
        Nat.1 + Nat.1 + Nat.1 + Nat.1 + Nat.1 = Nat.5
        directed_incidence_count(steps, KonigsbergVertex.v2) = Nat.5
        directed_incidence_count(steps, KonigsbergVertex.v3) = Nat.1 + Nat.1 + Nat.1
        directed_incidence_count(steps, KonigsbergVertex.v3) = Nat.3

        konigsberg_directed_degree_counts(steps) =
            (directed_incidence_count(steps, KonigsbergVertex.v0) = Nat.3 and
            directed_incidence_count(steps, KonigsbergVertex.v1) = Nat.3 and
            directed_incidence_count(steps, KonigsbergVertex.v2) = Nat.5 and
            directed_incidence_count(steps, KonigsbergVertex.v3) = Nat.3)
        konigsberg_directed_degree_counts(steps)
    }
}

/// Along a directed walk, departures and arrivals differ only at the two endpoints.
theorem directed_walk_source_target_balance(
    start: KonigsbergVertex, steps: List[DirectedKonigsbergBridge], end: KonigsbergVertex,
    v: KonigsbergVertex
) {
    directed_konigsberg_walk(start, steps, end) implies
        source_count(steps, v) + vertex_indicator(end, v) =
        target_count(steps, v) + vertex_indicator(start, v)
} by {
    define p(xs: List[DirectedKonigsbergBridge]) -> Bool {
        forall(s: KonigsbergVertex, t: KonigsbergVertex, w: KonigsbergVertex) {
            directed_konigsberg_walk(s, xs, t) implies
                source_count(xs, w) + vertex_indicator(t, w) =
                target_count(xs, w) + vertex_indicator(s, w)
        }
    }

    p(List.nil[DirectedKonigsbergBridge])

    forall(step: DirectedKonigsbergBridge, tail: List[DirectedKonigsbergBridge]) {
        if p(tail) {
            forall(s: KonigsbergVertex, t: KonigsbergVertex, w: KonigsbergVertex) {
                if directed_konigsberg_walk(s, List.cons(step, tail), t) {
                    directed_source(step) = s
                    directed_konigsberg_walk(directed_target(step), tail, t)
                    source_count(tail, w) + vertex_indicator(t, w) =
                        target_count(tail, w) + vertex_indicator(directed_target(step), w)
                    source_count(List.cons(step, tail), w) =
                        vertex_indicator(directed_source(step), w) + source_count(tail, w)
                    target_count(List.cons(step, tail), w) =
                        vertex_indicator(directed_target(step), w) + target_count(tail, w)
                    vertex_indicator(directed_source(step), w) = vertex_indicator(s, w)
                    vertex_indicator(directed_source(step), w) + source_count(tail, w) +
                        vertex_indicator(t, w) =
                        vertex_indicator(s, w) + target_count(tail, w) +
                        vertex_indicator(directed_target(step), w)
                    vertex_indicator(s, w) + target_count(tail, w) +
                        vertex_indicator(directed_target(step), w) =
                        vertex_indicator(directed_target(step), w) + target_count(tail, w) +
                        vertex_indicator(s, w)
                    source_count(List.cons(step, tail), w) + vertex_indicator(t, w) =
                        target_count(List.cons(step, tail), w) + vertex_indicator(s, w)
                }
            }
            p(List.cons(step, tail))
        }
    }

    p(steps)
}

/// A non-endpoint land mass has an even incident count along a directed walk.
theorem directed_walk_internal_incidence_double(
    start: KonigsbergVertex, steps: List[DirectedKonigsbergBridge], end: KonigsbergVertex,
    v: KonigsbergVertex
) {
    directed_konigsberg_walk(start, steps, end) and start != v and end != v implies
        exists(k: Nat) { directed_incidence_count(steps, v) = k + k }
} by {
    if directed_konigsberg_walk(start, steps, end) and start != v and end != v {
        directed_walk_source_target_balance(start, steps, end, v)
        source_count(steps, v) + vertex_indicator(end, v) =
            target_count(steps, v) + vertex_indicator(start, v)
        vertex_indicator(start, v) = Nat.0
        vertex_indicator(end, v) = Nat.0
        source_count(steps, v) + Nat.0 = target_count(steps, v) + Nat.0
        source_count(steps, v) = target_count(steps, v)
        let k = source_count(steps, v)
        directed_incidence_count(steps, v) = source_count(steps, v) + target_count(steps, v)
        directed_incidence_count(steps, v) = k + k
        exists(m: Nat) { directed_incidence_count(steps, v) = m + m }
    }
}

/// Twice a natural number is not odd.
theorem double_not_odd(k: Nat) {
    not odd_nat(k + k)
} by {
    define p(n: Nat) -> Bool {
        not odd_nat(n + n)
    }

    p(Nat.0)

    forall(n: Nat) {
        if p(n) {
            n.suc + n.suc = (n + n).suc.suc
            odd_nat((n + n).suc) = not odd_nat(n + n)
            odd_nat((n + n).suc.suc) = not odd_nat((n + n).suc)
            not odd_nat((n + n).suc) = odd_nat(n + n)
            not odd_nat(n.suc + n.suc)
            p(n.suc)
        }
    }

    p(k)
}

/// Three is not twice a natural number.
theorem not_double_eq_three(k: Nat) {
    k + k != Nat.3
} by {
    double_not_odd(k)
    if k + k = Nat.3 {
        odd_nat(k + k) = odd_nat(Nat.3)
        odd_nat(Nat.3)
        odd_nat(k + k)
        false
    }
}

/// Five is not twice a natural number.
theorem not_double_eq_five(k: Nat) {
    k + k != Nat.5
} by {
    double_not_odd(k)
    if k + k = Nat.5 {
        odd_nat(k + k) = odd_nat(Nat.5)
        odd_nat(Nat.5)
        odd_nat(k + k)
        false
    }
}

/// A vertex with incident count three in a directed walk must be an endpoint.
theorem incidence_three_forces_endpoint(
    start: KonigsbergVertex, steps: List[DirectedKonigsbergBridge], end: KonigsbergVertex,
    v: KonigsbergVertex
) {
    directed_konigsberg_walk(start, steps, end) and directed_incidence_count(steps, v) = Nat.3
        implies start = v or end = v
} by {
    if directed_konigsberg_walk(start, steps, end) and directed_incidence_count(steps, v) = Nat.3 {
        if not (start = v or end = v) {
            start != v
            end != v
            directed_walk_internal_incidence_double(start, steps, end, v)
            let k: Nat satisfy { directed_incidence_count(steps, v) = k + k }
            k + k = Nat.3
            not_double_eq_three(k)
            false
        }
    }
}

/// A vertex with incident count five in a directed walk must be an endpoint.
theorem incidence_five_forces_endpoint(
    start: KonigsbergVertex, steps: List[DirectedKonigsbergBridge], end: KonigsbergVertex,
    v: KonigsbergVertex
) {
    directed_konigsberg_walk(start, steps, end) and directed_incidence_count(steps, v) = Nat.5
        implies start = v or end = v
} by {
    if directed_konigsberg_walk(start, steps, end) and directed_incidence_count(steps, v) = Nat.5 {
        if not (start = v or end = v) {
            start != v
            end != v
            directed_walk_internal_incidence_double(start, steps, end, v)
            let k: Nat satisfy { directed_incidence_count(steps, v) = k + k }
            k + k = Nat.5
            not_double_eq_five(k)
            false
        }
    }
}

/// Two endpoints cannot contain all four Konigsberg land masses.
theorem four_vertices_not_two_endpoints(start: KonigsbergVertex, end: KonigsbergVertex) {
    ((endpoint_has(start, end, KonigsbergVertex.v0) and
        endpoint_has(start, end, KonigsbergVertex.v1)) and
    (endpoint_has(start, end, KonigsbergVertex.v2) and
        endpoint_has(start, end, KonigsbergVertex.v3))) implies false
} by {
    if (endpoint_has(start, end, KonigsbergVertex.v0) and
        endpoint_has(start, end, KonigsbergVertex.v1)) and
        (endpoint_has(start, end, KonigsbergVertex.v2) and
        endpoint_has(start, end, KonigsbergVertex.v3)) {
        start = KonigsbergVertex.v0 or end = KonigsbergVertex.v0
        start = KonigsbergVertex.v1 or end = KonigsbergVertex.v1
        start = KonigsbergVertex.v2 or end = KonigsbergVertex.v2
        start = KonigsbergVertex.v3 or end = KonigsbergVertex.v3
        match start {
            KonigsbergVertex.v0 {
                match end {
                    KonigsbergVertex.v0 {
                        start != KonigsbergVertex.v1
                        end != KonigsbergVertex.v1
                        false
                    }
                    KonigsbergVertex.v1 {
                        start != KonigsbergVertex.v2
                        end != KonigsbergVertex.v2
                        false
                    }
                    KonigsbergVertex.v2 {
                        start != KonigsbergVertex.v1
                        end != KonigsbergVertex.v1
                        false
                    }
                    KonigsbergVertex.v3 {
                        start != KonigsbergVertex.v1
                        end != KonigsbergVertex.v1
                        false
                    }
                }
            }
            KonigsbergVertex.v1 {
                match end {
                    KonigsbergVertex.v0 {
                        start != KonigsbergVertex.v2
                        end != KonigsbergVertex.v2
                        false
                    }
                    KonigsbergVertex.v1 {
                        start != KonigsbergVertex.v0
                        end != KonigsbergVertex.v0
                        false
                    }
                    KonigsbergVertex.v2 {
                        start != KonigsbergVertex.v0
                        end != KonigsbergVertex.v0
                        false
                    }
                    KonigsbergVertex.v3 {
                        start != KonigsbergVertex.v0
                        end != KonigsbergVertex.v0
                        false
                    }
                }
            }
            KonigsbergVertex.v2 {
                match end {
                    KonigsbergVertex.v0 {
                        start != KonigsbergVertex.v1
                        end != KonigsbergVertex.v1
                        false
                    }
                    KonigsbergVertex.v1 {
                        start != KonigsbergVertex.v0
                        end != KonigsbergVertex.v0
                        false
                    }
                    KonigsbergVertex.v2 {
                        start != KonigsbergVertex.v0
                        end != KonigsbergVertex.v0
                        false
                    }
                    KonigsbergVertex.v3 {
                        start != KonigsbergVertex.v0
                        end != KonigsbergVertex.v0
                        false
                    }
                }
            }
            KonigsbergVertex.v3 {
                match end {
                    KonigsbergVertex.v0 {
                        start != KonigsbergVertex.v1
                        end != KonigsbergVertex.v1
                        false
                    }
                    KonigsbergVertex.v1 {
                        start != KonigsbergVertex.v0
                        end != KonigsbergVertex.v0
                        false
                    }
                    KonigsbergVertex.v2 {
                        start != KonigsbergVertex.v0
                        end != KonigsbergVertex.v0
                        false
                    }
                    KonigsbergVertex.v3 {
                        start != KonigsbergVertex.v0
                        end != KonigsbergVertex.v0
                        false
                    }
                }
            }
        }
    }
}

/// The Konigsberg bridge graph has no Eulerian trail.
theorem top100_054_konigsberg_bridges(steps: List[DirectedKonigsbergBridge]) {
    not directed_konigsberg_eulerian_trail(steps)
} by {
    if directed_konigsberg_eulerian_trail(steps) {
        directed_konigsberg_eulerian_trail(steps) =
            (directed_uses_every_bridge_once(steps) and
                exists(start: KonigsbergVertex, end: KonigsbergVertex) {
                    directed_konigsberg_walk(start, steps, end)
                })
        directed_uses_every_bridge_once(steps)
        directed_uses_every_bridge_once_imp_degree_counts(steps)
        konigsberg_directed_degree_counts(steps)
        konigsberg_directed_degree_counts(steps) =
            (directed_incidence_count(steps, KonigsbergVertex.v0) = Nat.3 and
            directed_incidence_count(steps, KonigsbergVertex.v1) = Nat.3 and
            directed_incidence_count(steps, KonigsbergVertex.v2) = Nat.5 and
            directed_incidence_count(steps, KonigsbergVertex.v3) = Nat.3)
        directed_incidence_count(steps, KonigsbergVertex.v0) = Nat.3
        directed_incidence_count(steps, KonigsbergVertex.v1) = Nat.3
        directed_incidence_count(steps, KonigsbergVertex.v2) = Nat.5
        directed_incidence_count(steps, KonigsbergVertex.v3) = Nat.3
        let (start: KonigsbergVertex, end: KonigsbergVertex) satisfy {
            directed_konigsberg_walk(start, steps, end)
        }

        incidence_three_forces_endpoint(start, steps, end, KonigsbergVertex.v0)
        incidence_three_forces_endpoint(start, steps, end, KonigsbergVertex.v1)
        incidence_five_forces_endpoint(start, steps, end, KonigsbergVertex.v2)
        incidence_three_forces_endpoint(start, steps, end, KonigsbergVertex.v3)

        start = KonigsbergVertex.v0 or end = KonigsbergVertex.v0
        start = KonigsbergVertex.v1 or end = KonigsbergVertex.v1
        start = KonigsbergVertex.v2 or end = KonigsbergVertex.v2
        start = KonigsbergVertex.v3 or end = KonigsbergVertex.v3
        endpoint_has(start, end, KonigsbergVertex.v0)
        endpoint_has(start, end, KonigsbergVertex.v1)
        endpoint_has(start, end, KonigsbergVertex.v2)
        endpoint_has(start, end, KonigsbergVertex.v3)
        endpoint_has(start, end, KonigsbergVertex.v0) and endpoint_has(start, end, KonigsbergVertex.v1)
        endpoint_has(start, end, KonigsbergVertex.v2) and endpoint_has(start, end, KonigsbergVertex.v3)
        (endpoint_has(start, end, KonigsbergVertex.v0) and endpoint_has(start, end, KonigsbergVertex.v1)) and
            (endpoint_has(start, end, KonigsbergVertex.v2) and endpoint_has(start, end, KonigsbergVertex.v3))

        four_vertices_not_two_endpoints(start, end)
        false
    }
}
