from int import Int, add_zero_left
from data.int.int_digits import int_digit_sum, int_read_digits_acc, int_read_digits_acc_mod3_digit_sum
from list import List
from nat import Nat
from data.basic.relation_basic import equivalence_flip, equivalence_step
from zmod import int_mod_rel, int_mod_rel_is_equivalence, int_mod_rel_zero_iff_int_three_divides

/// AixBot-generated Top 100 #85: divisibility by 3 is preserved by taking the digit sum.
theorem top100_085_divisibility_by_3_rule(digits: List[Int]) {
    Int.3.divides(int_read_digits_acc(digits, Int.0)) =
    Int.3.divides(int_digit_sum(digits))
} by {
    let n = int_read_digits_acc(digits, Int.0)
    let s = int_digit_sum(digits)

    int_read_digits_acc_mod3_digit_sum(digits, Int.0)
    add_zero_left(s)
    int_mod_rel(Nat.3, n, s)

    int_mod_rel_is_equivalence(Nat.3)
    if int_mod_rel(Nat.3, n, Int.0) {
        equivalence_flip[Int](int_mod_rel(Nat.3), n, s)
        int_mod_rel(Nat.3, s, n)
        equivalence_step[Int](int_mod_rel(Nat.3), s, n, Int.0)
        int_mod_rel(Nat.3, s, Int.0)
    }
    if int_mod_rel(Nat.3, s, Int.0) {
        equivalence_step[Int](int_mod_rel(Nat.3), n, s, Int.0)
        int_mod_rel(Nat.3, n, Int.0)
    }
    int_mod_rel(Nat.3, n, Int.0) = int_mod_rel(Nat.3, s, Int.0)

    int_mod_rel_zero_iff_int_three_divides(n)
    int_mod_rel_zero_iff_int_three_divides(s)
    Int.3.divides(n) = Int.3.divides(s)
}
