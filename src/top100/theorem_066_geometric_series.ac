from list import map
from list import sum
from nat import Nat
from algebra.ring.ring import Ring, geometric_sum

/// Freek Top 100 #66: the finite sum formula for a geometric series.
theorem top100_066_geometric_series[R: Ring](a: R, n: Nat) {
    (a + -R.1) * sum(map(n.range, a.pow)) = a.pow(n) + -R.1
} by {
    geometric_sum[R](a, n)
}
