from algebra.field.field import mul_not_zero, unique_inverse
from list import partial
from nat import Nat, lte_trans
from real import mul_one_over
from real import harmonic, harmonic_block_term_lower, harmonic_nonnegative
from real import Real, close_comm, close_imp_bounds, one_half_plus_one_half
from real import div_mul_cancel_right
from real import real_mul_comm
from real import cauchy_bound, converges
from real import is_lower_bound
from nat import from_nat, from_nat_zero, from_nat_add

/// Splitting a real partial sum into its first `n` terms and the following `m` terms.
theorem partial_add_shift_real(f: Nat -> Real, n: Nat, m: Nat) {
    partial[Real](f, n + m) = partial[Real](f, n) + partial[Real](function(k: Nat) { f(n + k) }, m)
} by {
    define g(k: Nat) -> Real { f(n + k) }
    define p(x: Nat) -> Bool {
        partial[Real](f, n + x) = partial[Real](f, n) + partial[Real](g, x)
    }
    partial[Real](g, Nat.0) = Real.0
    partial[Real](f, n + Nat.0) = partial[Real](f, n)
    p(Nat.0)
    forall(x: Nat) {
        if p(x) {
            n + x.suc = (n + x).suc
            partial[Real](f, n + x.suc) = partial[Real](f, (n + x).suc)
            partial[Real](f, (n + x).suc) = partial[Real](f, n + x) + f(n + x)
            partial[Real](f, n + x.suc) = partial[Real](f, n + x) + f(n + x)
            partial[Real](f, n + x) + f(n + x) = (partial[Real](f, n) + partial[Real](g, x)) + f(n + x)
            (partial[Real](f, n) + partial[Real](g, x)) + f(n + x) = partial[Real](f, n) + (partial[Real](g, x) + f(n + x))
            partial[Real](g, x.suc) = partial[Real](g, x) + g(x)
            g(x) = f(n + x)
            partial[Real](g, x.suc) = partial[Real](g, x) + f(n + x)
            partial[Real](f, n) + (partial[Real](g, x) + f(n + x)) = partial[Real](f, n) + partial[Real](g, x.suc)
            p(x.suc)
        }
    }
    p(m)
    forall(k: Nat) { g(k) = f(n + k) }
    partial[Real](g, m) = partial[Real](function(k: Nat) { f(n + k) }, m)
}

/// Adding two lower bounds gives a lower bound on the sum.
theorem real_add_two_lower(a: Real, b: Real, u: Real, v: Real) {
    a >= u and b >= v implies a + b >= u + v
} by {
    a + b >= u + b
    u + b >= u + v
    a + b >= u + v
}

/// Multiplication by a reciprocal respects successor expansion of natural reals.
theorem from_nat_suc_mul_recip_step(x: Nat, d: Nat) {
    from_nat[Real](x.suc) * (Real.1 / from_nat[Real](d)) = from_nat[Real](x) * (Real.1 / from_nat[Real](d)) + (Real.1 / from_nat[Real](d))
} by {
    mul_one_over(Real.1, Real.1)
    x.suc = x + Nat.1
    from_nat[Real](x.suc) = from_nat[Real](x + Nat.1)
    from_nat[Real](x + Nat.1) = from_nat[Real](x) + from_nat[Real](Nat.1)
    from_nat[Real](Nat.1) = Real.1
    from_nat[Real](x.suc) = from_nat[Real](x) + Real.1
    from_nat[Real](x.suc) * (Real.1 / from_nat[Real](d)) = (from_nat[Real](x) + Real.1) * (Real.1 / from_nat[Real](d))
    (from_nat[Real](x) + Real.1) * (Real.1 / from_nat[Real](d)) = from_nat[Real](x) * (Real.1 / from_nat[Real](d)) + Real.1 * (Real.1 / from_nat[Real](d))
    Real.1 * (Real.1 / from_nat[Real](d)) = Real.1 / from_nat[Real](d)
}

/// A partial sum with each term above a common reciprocal grows by that reciprocal.
theorem partial_real_lower_step_generic(f: Nat -> Real, x: Nat, d: Nat) {
    partial[Real](f, x) >= from_nat[Real](x) * (Real.1 / from_nat[Real](d)) and
    f(x) >= Real.1 / from_nat[Real](d) implies
    partial[Real](f, x.suc) >= from_nat[Real](x.suc) * (Real.1 / from_nat[Real](d))
} by {
    partial[Real](f, x.suc) = partial[Real](f, x) + f(x)
    real_add_two_lower(partial[Real](f, x), f(x), from_nat[Real](x) * (Real.1 / from_nat[Real](d)), Real.1 / from_nat[Real](d))
    partial[Real](f, x) + f(x) >= from_nat[Real](x) * (Real.1 / from_nat[Real](d)) + (Real.1 / from_nat[Real](d))
    from_nat_suc_mul_recip_step(x, d)
    from_nat[Real](x.suc) * (Real.1 / from_nat[Real](d)) = from_nat[Real](x) * (Real.1 / from_nat[Real](d)) + (Real.1 / from_nat[Real](d))
}

/// Twice one half is one.
theorem two_mul_one_half_real {
    (Real.1 + Real.1) * Real.one_half = Real.1
} by {
    real_mul_comm(Real.one_half, Real.1)
    (Real.1 + Real.1) * Real.one_half = Real.1 * Real.one_half + Real.1 * Real.one_half
    Real.1 * Real.one_half = Real.one_half
    (Real.1 + Real.1) * Real.one_half = Real.one_half + Real.one_half
    one_half_plus_one_half
}

/// Multiplying by two and then by one half gives the original real.
theorem two_times_a_half(a: Real) {
    ((Real.1 + Real.1) * a) * Real.one_half = a
} by {
    real_mul_comm(Real.1, a)
    let two: Real = Real.1 + Real.1
    let half: Real = Real.one_half
    (two * a) * half = two * (a * half)
    two * (a * half) = a * (two * half)
    two * half = Real.1
    a * (two * half) = a
}

/// Cancelling a common nonzero factor in a quotient.
theorem touch_field(a: Real, b: Real) {
    a != Real.0 and b != Real.0 implies a / (b * a) = Real.1 / b
} by {
    div_mul_cancel_right(a, b)
}

/// Multiplying a nonzero real by two gives a nonzero real.
theorem two_mul_nonzero_real(a: Real) {
    a != Real.0 implies (Real.1 + Real.1) * a != Real.0
} by {
    real_mul_comm(Real.1 + Real.1, a)
    Real.1 + Real.1 != Real.0
    mul_not_zero[Real](Real.1 + Real.1, a)
}

/// One half is the inverse of two.
theorem half_is_inverse_two {
    Real.one_half = (Real.1 + Real.1).inverse
} by {
    (Real.1 + Real.1) * Real.one_half = Real.1
    unique_inverse[Real](Real.1 + Real.1, Real.one_half)
}

/// The reciprocal of two is one half.
theorem real_one_div_two_eq_half {
    Real.1 / (Real.1 + Real.1) = Real.one_half
} by {
    half_is_inverse_two
    Real.1 / (Real.1 + Real.1) = Real.1 * (Real.1 + Real.1).inverse
    Real.1 * (Real.1 + Real.1).inverse = (Real.1 + Real.1).inverse
}

/// A nonzero real divided by twice itself is one half.
theorem real_half_ratio_nonzero(a: Real) {
    a != Real.0 implies a * (Real.1 / ((Real.1 + Real.1) * a)) = Real.one_half
} by {
    let two: Real = Real.1 + Real.1
    two != Real.0
    two * a != Real.0
    a * (Real.1 / (two * a)) = a / (two * a)
    a / (two * a) = Real.1 / two
    Real.1 / two = Real.one_half
}

/// A positive successor natural is one half of its double as a reciprocal product.
theorem from_nat_suc_half_ratio(k: Nat) {
    from_nat[Real](k.suc) * (Real.1 / from_nat[Real](k.suc + k.suc)) = Real.one_half
} by {
    let a: Real = from_nat[Real](k.suc)
    from_nat[Real](k.suc + k.suc) = from_nat[Real](k.suc) + from_nat[Real](k.suc)
    from_nat[Real](k.suc + k.suc) = a + a
    a + a = (Real.1 + Real.1) * a
    from_nat[Real](k.suc + k.suc) = (Real.1 + Real.1) * a
    a != Real.0
    a * (Real.1 / ((Real.1 + Real.1) * a)) = Real.one_half
    from_nat[Real](k.suc) * (Real.1 / from_nat[Real](k.suc + k.suc)) = Real.one_half
}

/// A nonzero natural real is one half of its double as a reciprocal product.
theorem from_nat_half_ratio_ne_zero(n: Nat) {
    n != Nat.0 implies from_nat[Real](n) * (Real.1 / from_nat[Real](n + n)) = Real.one_half
} by {
    define p(x: Nat) -> Bool {
        x != Nat.0 implies from_nat[Real](x) * (Real.1 / from_nat[Real](x + x)) = Real.one_half
    }
    if Nat.0 != Nat.0 {
        false
    }
    p(Nat.0)
    forall(k: Nat) {
        if p(k) {
            from_nat_suc_half_ratio(k)
            p(k.suc)
        }
    }
    p(n)
}

/// A positive natural is nonzero.
theorem nat_pos_ne_zero_atomic(n: Nat) {
    n > Nat.0 implies n != Nat.0
}

/// A positive natural real is one half of its double as a reciprocal product.
theorem from_nat_half_ratio(n: Nat) {
    n > Nat.0 implies from_nat[Real](n) * (Real.1 / from_nat[Real](n + n)) = Real.one_half
} by {
    nat_pos_ne_zero_atomic(n)
    from_nat_half_ratio_ne_zero(n)
}

/// Powers of two are positive.
theorem nat_two_pow_pos(t: Nat) {
    Nat.2.pow(t) > Nat.0
}

/// A successor power of two is the double of the previous power.
theorem nat_two_pow_suc_eq_double(t: Nat) {
    Nat.2.pow(t.suc) = Nat.2.pow(t) + Nat.2.pow(t)
} by {
    Nat.2.pow(t.suc) = Nat.2 * Nat.2.pow(t)
    Nat.2 * Nat.2.pow(t) = Nat.2.pow(t) * Nat.2
    Nat.2.pow(t) * Nat.2 = Nat.2.pow(t) + Nat.2.pow(t)
}

/// Natural successor as a real adds one.
theorem from_nat_suc_eq_add_one_real(t: Nat) {
    from_nat[Real](t.suc) = from_nat[Real](t) + Real.1
} by {
    t.suc = t + Nat.1
    from_nat[Real](t.suc) = from_nat[Real](t + Nat.1)
    from_nat[Real](t + Nat.1) = from_nat[Real](t) + from_nat[Real](Nat.1)
    from_nat[Real](Nat.1) = Real.1
    from_nat[Real](t.suc) = from_nat[Real](t) + Real.1
}

/// Multiplication by one half respects successor expansion of natural reals.
theorem from_nat_suc_mul_half_step(t: Nat) {
    from_nat[Real](t.suc) * Real.one_half = from_nat[Real](t) * Real.one_half + Real.one_half
} by {
    from_nat_suc_eq_add_one_real(t)
    from_nat[Real](t.suc) = from_nat[Real](t) + Real.1
    from_nat[Real](t.suc) * Real.one_half = (from_nat[Real](t) + Real.1) * Real.one_half
    (from_nat[Real](t) + Real.1) * Real.one_half = from_nat[Real](t) * Real.one_half + Real.1 * Real.one_half
    Real.1 * Real.one_half = Real.one_half
}

/// Extending a harmonic block by one preserves the reciprocal block lower bound.
theorem harmonic_block_partial_lower_step(n: Nat, x: Nat) {
    n > Nat.0 and x.suc <= n and
    partial[Real](function(k: Nat) { harmonic(n + k) }, x) >= from_nat[Real](x) * (Real.1 / from_nat[Real](n + n)) implies
    partial[Real](function(k: Nat) { harmonic(n + k) }, x.suc) >= from_nat[Real](x.suc) * (Real.1 / from_nat[Real](n + n))
} by {
    x < n
    harmonic_block_term_lower(n, x)
    partial_real_lower_step_generic(function(k: Nat) { harmonic(n + k) }, x, n + n)
}

/// The first `m` terms of the dyadic harmonic block are bounded below by `m / (2n)`.
theorem harmonic_block_partial_lower(n: Nat, m: Nat) {
    n > Nat.0 and m <= n implies
    partial[Real](function(k: Nat) { harmonic(n + k) }, m) >= from_nat[Real](m) * (Real.1 / from_nat[Real](n + n))
} by {
    define p(x: Nat) -> Bool {
        n > Nat.0 and x <= n implies partial[Real](function(k: Nat) { harmonic(n + k) }, x) >= from_nat[Real](x) * (Real.1 / from_nat[Real](n + n))
    }
    partial[Real](function(k: Nat) { harmonic(n + k) }, Nat.0) = Real.0
    from_nat[Real](Nat.0) = Real.0
    from_nat[Real](Nat.0) * (Real.1 / from_nat[Real](n + n)) = Real.0
    p(Nat.0)
    forall(x: Nat) {
        if p(x) {
            if n > Nat.0 and x.suc <= n {
                x <= n
                partial[Real](function(k: Nat) { harmonic(n + k) }, x) >= from_nat[Real](x) * (Real.1 / from_nat[Real](n + n))
                harmonic_block_partial_lower_step(n, x)
                partial[Real](function(k: Nat) { harmonic(n + k) }, x.suc) >= from_nat[Real](x.suc) * (Real.1 / from_nat[Real](n + n))
            }
            p(x.suc)
        }
    }
    p(m)
}

/// A full dyadic harmonic block has total at least one half.
theorem harmonic_block_lower(n: Nat) {
    n > Nat.0 implies partial[Real](function(k: Nat) { harmonic(n + k) }, n) >= Real.one_half
} by {
    harmonic_block_partial_lower(n, n)
    from_nat_half_ratio(n)
}

/// Harmonic terms are nonnegative.
theorem harmonic_is_lower_bound {
    is_lower_bound(harmonic, Real.0)
} by {
    forall(k: Nat) {
        harmonic_nonnegative(k)
    }
}

/// The dyadic lower bound advances from `2^t` to `2^(t+1)`.
theorem harmonic_dyadic_lower_step(t: Nat) {
    partial[Real](harmonic, Nat.2.pow(t)) >= from_nat[Real](t) * Real.one_half implies
    partial[Real](harmonic, Nat.2.pow(t.suc)) >= from_nat[Real](t.suc) * Real.one_half
} by {
    let n: Nat = Nat.2.pow(t)
    n > Nat.0
    partial[Real](function(k: Nat) { harmonic(n + k) }, n) >= Real.one_half
    partial[Real](harmonic, n + n) = partial[Real](harmonic, n) + partial[Real](function(k: Nat) { harmonic(n + k) }, n)
    partial[Real](harmonic, n) + partial[Real](function(k: Nat) { harmonic(n + k) }, n) >= from_nat[Real](t) * Real.one_half + Real.one_half
    from_nat[Real](t.suc) * Real.one_half = from_nat[Real](t) * Real.one_half + Real.one_half
    partial[Real](harmonic, n + n) >= from_nat[Real](t.suc) * Real.one_half
    Nat.2.pow(t.suc) = n + n
    partial[Real](harmonic, Nat.2.pow(t.suc)) >= from_nat[Real](t.suc) * Real.one_half
}

/// The dyadic lower bound holds at the base index.
theorem harmonic_dyadic_lower_base {
    partial[Real](harmonic, Nat.2.pow(Nat.0)) >= from_nat[Real](Nat.0) * Real.one_half
} by {
    is_lower_bound(harmonic, Real.0)
    partial[Real](harmonic, Nat.2.pow(Nat.0)) >= Real.0
    from_nat[Real](Nat.0) = Real.0
    from_nat[Real](Nat.0) * Real.one_half = Real.0
}

/// Dyadic partial sums of the harmonic series are bounded below by `t/2`.
theorem harmonic_dyadic_lower_point(t: Nat) {
    partial[Real](harmonic, Nat.2.pow(t)) >= from_nat[Real](t) * Real.one_half
} by {
    define f(x: Nat) -> Bool {
        partial[Real](harmonic, Nat.2.pow(x)) >= from_nat[Real](x) * Real.one_half
    }
    f(Nat.0)
    forall(x: Nat) {
        if f(x) {
            harmonic_dyadic_lower_step(x)
            f(x.suc)
        }
    }
    f(t)
}

/// Doubling any positive partial-sum index increases the harmonic partial sum by at least one half.
theorem harmonic_tail_half_jump(k: Nat) {
    k > Nat.0 implies partial[Real](harmonic, k + k) >= partial[Real](harmonic, k) + Real.one_half
} by {
    partial_add_shift_real(harmonic, k, k)
    partial[Real](harmonic, k + k) = partial[Real](harmonic, k) + partial[Real](function(t: Nat) { harmonic(k + t) }, k)
    harmonic_block_lower(k)
    partial[Real](function(t: Nat) { harmonic(k + t) }, k) >= Real.one_half
    partial[Real](harmonic, k) + partial[Real](function(t: Nat) { harmonic(k + t) }, k) >= partial[Real](harmonic, k) + Real.one_half
}

/// One half is positive.
theorem real_one_half_is_positive {
    Real.one_half.is_positive
} by {
    Real.0 < Real.one_half
}

/// A convergent real sequence has a Cauchy bound at radius one half.
theorem converges_has_half_cauchy_bound(q: Nat -> Real) {
    converges(q) implies exists(n: Nat) { cauchy_bound(q, n, Real.one_half) }
} by {
    if converges(q) {
        converges(q) = forall(eps: Real) {
            eps.is_positive implies exists(n: Nat) { cauchy_bound(q, n, eps) }
        }
        forall(eps: Real) {
            eps.is_positive implies exists(n: Nat) { cauchy_bound(q, n, eps) }
        }
        Real.one_half.is_positive
        exists(n: Nat) { cauchy_bound(q, n, Real.one_half) }
    }
}

/// A Cauchy bound controls every pair of indices beyond the bound.
theorem cauchy_bound_all_indices(q: Nat -> Real, n: Nat, eps: Real) {
    cauchy_bound(q, n, eps) implies forall(i: Nat, j: Nat) {
        n <= i and n <= j implies q(i).is_close(q(j), eps)
    }
}

/// A Cauchy bound controls a chosen pair of indices beyond the bound.
theorem cauchy_bound_indices(q: Nat -> Real, n: Nat, eps: Real, i: Nat, j: Nat) {
    cauchy_bound(q, n, eps) and n <= i and n <= j implies q(i).is_close(q(j), eps)
} by {
    cauchy_bound_all_indices(q, n, eps)
    forall(a: Nat, b: Nat) {
        n <= a and n <= b implies q(a).is_close(q(b), eps)
    }
    q(i).is_close(q(j), eps)
}

/// A gap of at least one half is not one-half-close.
theorem not_close_if_ge_half(a: Real, b: Real) {
    b >= a + Real.one_half implies not b.is_close(a, Real.one_half)
} by {
    if b.is_close(a, Real.one_half) {
        close_imp_bounds(b, a, Real.one_half)
        b < a + Real.one_half
        false
    }
}

/// AixBot-generated Freek Top 100 #34: the harmonic series does not converge.
theorem top100_034_divergence_of_harmonic_series {
    not converges(partial[Real](harmonic))
} by {
    if converges(partial[Real](harmonic)) {
        Real.one_half.is_positive
        converges_has_half_cauchy_bound(partial[Real](harmonic))
        let n: Nat satisfy {
            cauchy_bound(partial[Real](harmonic), n, Real.one_half)
        }
        let k: Nat = n.suc
        k > Nat.0
        n <= k
        n <= k + k
        harmonic_tail_half_jump(k)
        partial[Real](harmonic, k + k) >= partial[Real](harmonic, k) + Real.one_half
        not_close_if_ge_half(partial[Real](harmonic, k), partial[Real](harmonic, k + k))
        cauchy_bound_indices(partial[Real](harmonic), n, Real.one_half, k, k + k)
        partial[Real](harmonic, k).is_close(partial[Real](harmonic, k + k), Real.one_half)
        close_comm(partial[Real](harmonic, k), partial[Real](harmonic, k + k), Real.one_half)
        partial[Real](harmonic, k + k).is_close(partial[Real](harmonic, k), Real.one_half)
        false
    }
}
