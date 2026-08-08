from algebra.field.field import mul_not_zero
from nat import Nat
from rat import Rat
from real import suc_pos
from real import Real
from real import inverse_div
from real import from_nat_is_from_rat
from nat import from_nat

/// AixBot-generated Freek Top 100 #42: sum of the reciprocals of the triangular numbers.
theorem from_nat_suc_ne_zero_real(n: Nat) { not from_nat[Real](n.suc) = Real.0 } by {
    from_nat_is_from_rat(n.suc)
    suc_pos(n)
    from_nat[Real](n.suc) = Real.from_rat(Rat.from_nat(n.suc))
    Real.from_rat(Rat.from_nat(n.suc)) > Real.0
    from_nat[Real](n.suc) > Real.0
}

/// The reciprocal of a quotient is the reversed quotient.
theorem one_div_div_real(a: Real, b: Real) {
    a != Real.0 and b != Real.0 implies Real.1 / (a / b) = b / a
} by {
    inverse_div(a, b)
    (a / b).inverse = b / a
    Real.1 / (a / b) = (a / b).inverse
    Real.1 / (a / b) = b / a
}

/// The product of two successive positive natural reals is nonzero.
theorem from_nat_suc_mul_suc_ne_zero_real(k: Nat) {
    not from_nat[Real](k.suc) * from_nat[Real](k.suc.suc) = Real.0
} by {
    from_nat_suc_ne_zero_real(k)
    from_nat_suc_ne_zero_real(k.suc)
    mul_not_zero[Real](from_nat[Real](k.suc), from_nat[Real](k.suc.suc))
}

/// Two is nonzero in the reals.
theorem two_ne_zero_real { not (Real.1 + Real.1) = Real.0 }

/// The difference of two reciprocals is the difference over the product.
theorem real_one_div_sub_one_div(a: Real, b: Real) {
    a != Real.0 and b != Real.0 implies Real.1 / a - Real.1 / b = (b - a) / (a * b)
} by {
    mul_not_zero[Real](a, b)
    a * b != Real.0
    Real.1 / a = b / (a * b)
    Real.1 / b = a / (a * b)
    Real.1 / a - Real.1 / b = b / (a * b) - a / (a * b)
    b / (a * b) - a / (a * b) = (b - a) / (a * b)
}

/// Adjacent reciprocal differences give the reciprocal of the adjacent product.
theorem adjacent_reciprocal_difference(k: Nat) {
    Real.1 / from_nat[Real](k.suc) - Real.1 / from_nat[Real](k.suc.suc) =
    Real.1 / (from_nat[Real](k.suc) * from_nat[Real](k.suc.suc))
} by {
    let a: Real = from_nat[Real](k.suc)
    let b: Real = from_nat[Real](k.suc.suc)
    a != Real.0
    b != Real.0
    b = a + Real.1
    Real.1 / a - Real.1 / b = (b - a) / (a * b)
    b - a = Real.1
    (b - a) / (a * b) = Real.1 / (a * b)
}

/// A triangular-number reciprocal is twice an adjacent reciprocal difference.
theorem triangular_reciprocal_simplification(k: Nat) {
    Real.1 / ((from_nat[Real](k.suc) * from_nat[Real](k.suc.suc)) / (Real.1 + Real.1)) =
    (Real.1 + Real.1) *
        (Real.1 / from_nat[Real](k.suc) - Real.1 / from_nat[Real](k.suc.suc))
} by {
    let real_two: Real = Real.1 + Real.1
    let p: Real = from_nat[Real](k.suc) * from_nat[Real](k.suc.suc)
    not p = Real.0
    not real_two = Real.0
    Real.1 / (p / real_two) = real_two / p
    real_two / p = real_two * (Real.1 / p)
    Real.1 / from_nat[Real](k.suc) - Real.1 / from_nat[Real](k.suc.suc) = Real.1 / p
    real_two * (Real.1 / p) =
        real_two * (Real.1 / from_nat[Real](k.suc) - Real.1 / from_nat[Real](k.suc.suc))
}

/// Adjacent scaled differences telescope over the reals.
theorem real_telescoping_cancel(a: Real, b: Real) {
    (Real.1 + Real.1) * (Real.1 - a) + (Real.1 + Real.1) * (a - b) =
    (Real.1 + Real.1) * (Real.1 - b)
} by {
    let two: Real = Real.1 + Real.1
    Real.1 - a = Real.1 + -a
    a - b = a + -b
    Real.1 - b = Real.1 + -b
    (Real.1 - a) + (a - b) = (Real.1 + -a) + (a + -b)
    (Real.1 + -a) + (a + -b) = Real.1 + (-a + a) + -b
    -a + a = Real.0
    Real.1 + (-a + a) + -b = Real.1 + -b
    (Real.1 - a) + (a - b) = Real.1 - b
    two * (Real.1 - a) + two * (a - b) = two * ((Real.1 - a) + (a - b))
    two * ((Real.1 - a) + (a - b)) = two * (Real.1 - b)
}

/// The reciprocal of the kth triangular number, indexed from zero.
define triangular_term(k: Nat) -> Real {
    Real.1 / ((from_nat[Real](k.suc) * from_nat[Real](k.suc.suc)) / (Real.1 + Real.1))
}

/// The finite sum of reciprocal triangular numbers up to n.
define triangular_reciprocal_sum(n: Nat) -> Real {
    match n {
        Nat.zero { Real.0 }
        Nat.suc(m) { triangular_reciprocal_sum(m) + triangular_term(m) }
    }
}

/// The closed form for the partial sum is preserved by the next triangular term.
theorem finite_telescoping_step(x: Nat) {
    triangular_reciprocal_sum(x) =
        (Real.1 + Real.1) * (Real.1 - Real.1 / from_nat[Real](x.suc))
    implies
    triangular_reciprocal_sum(x.suc) =
        (Real.1 + Real.1) * (Real.1 - Real.1 / from_nat[Real](x.suc.suc))
} by {
    triangular_reciprocal_sum(x.suc) = triangular_reciprocal_sum(x) + triangular_term(x)
    triangular_term(x) =
        (Real.1 + Real.1) *
            (Real.1 / from_nat[Real](x.suc) - Real.1 / from_nat[Real](x.suc.suc))
    triangular_reciprocal_sum(x.suc) =
        (Real.1 + Real.1) * (Real.1 - Real.1 / from_nat[Real](x.suc)) +
        (Real.1 + Real.1) *
            (Real.1 / from_nat[Real](x.suc) - Real.1 / from_nat[Real](x.suc.suc))
    real_telescoping_cancel(Real.1 / from_nat[Real](x.suc), Real.1 / from_nat[Real](x.suc.suc))
    triangular_reciprocal_sum(x.suc) =
        (Real.1 + Real.1) * (Real.1 - Real.1 / from_nat[Real](x.suc.suc))
}

/// The closed form holds for the empty triangular reciprocal sum.
theorem finite_telescoping_base {
    triangular_reciprocal_sum(Nat.0) =
    (Real.1 + Real.1) * (Real.1 - Real.1 / from_nat[Real](Nat.1))
} by {
    triangular_reciprocal_sum(Nat.0) = Real.0
    from_nat[Real](Nat.1) = Real.1
    Real.1 / from_nat[Real](Nat.1) = Real.1
    Real.1 - Real.1 / from_nat[Real](Nat.1) = Real.0
    (Real.1 + Real.1) * (Real.1 - Real.1 / from_nat[Real](Nat.1)) = Real.0
}

/// The finite triangular reciprocal sum has the telescoping closed form.
theorem finite_telescoping_partial_sum(n: Nat) {
    triangular_reciprocal_sum(n) =
    (Real.1 + Real.1) * (Real.1 - Real.1 / from_nat[Real](n.suc))
} by {
    define p(x: Nat) -> Bool {
        triangular_reciprocal_sum(x) =
        (Real.1 + Real.1) * (Real.1 - Real.1 / from_nat[Real](x.suc))
    }
    finite_telescoping_base
    p(Nat.0)
    forall(x: Nat) {
        if p(x) {
            finite_telescoping_step(x)
            p(x.suc)
        }
    }
    p(n)
}

/// The telescoping closed form is the standard rational expression.
theorem telescoping_closed_form_algebra(n: Nat) {
    (Real.1 + Real.1) * (Real.1 - Real.1 / from_nat[Real](n.suc)) =
    ((Real.1 + Real.1) * from_nat[Real](n)) / from_nat[Real](n.suc)
} by {
    let d: Real = from_nat[Real](n.suc)
    let a: Real = from_nat[Real](n)
    not d = Real.0
    d = a + Real.1
    d - Real.1 = a
    d * (Real.1 - Real.1 / d) = d * Real.1 - d * (Real.1 / d)
    d * Real.1 = d
    d * (Real.1 / d) = Real.1
    d * (Real.1 - Real.1 / d) = d - Real.1
    d - Real.1 = d * (Real.1 - Real.1 / d)
    (d - Real.1) / d = Real.1 - Real.1 / d
    Real.1 - Real.1 / d = (d - Real.1) / d
    Real.1 - Real.1 / d = a / d
    (Real.1 + Real.1) * (Real.1 - Real.1 / d) = (Real.1 + Real.1) * (a / d)
    (Real.1 + Real.1) * (a / d) = ((Real.1 + Real.1) * a) / d
}

/// The finite sum of reciprocal triangular numbers has the Top 100 closed form.
theorem top100_042_triangular_reciprocal_sum(n: Nat) {
    triangular_reciprocal_sum(n) = ((Real.1 + Real.1) * from_nat[Real](n)) / from_nat[Real](n.suc)
} by {
    finite_telescoping_partial_sum(n)
    telescoping_closed_form_algebra(n)
}
