from list import List
from data.basic.set import Set

/// The auxiliary set list union object used in Lagrange's theorem.
define set_list_union[K](items: List[Set[K]]) -> Set[K] {
    match items {
        List.nil {
            Set[K].empty_set
        }
        List.cons(head, tail) {
            head.union(set_list_union[K](tail))
        }
    }
}

/// An auxiliary proposition for Lagrange's theorem.
theorem set_list_union_tail_from_ih_and_tail_contains[K](tail: List[Set[K]], c: Set[K], x: K) { (tail.contains(c) and c.contains(x) implies set_list_union[K](tail).contains(x)) implies (tail.contains(c) and c.contains(x) implies set_list_union[K](tail).contains(x)) }

/// An auxiliary proposition for Lagrange's theorem.
theorem set_list_union_contains_nil_member[K](c: Set[K], x: K) { List.nil[Set[K]].contains(c) and c.contains(x) implies set_list_union[K](List.nil[Set[K]]).contains(x) } by {
    false
}

/// An auxiliary proposition for Lagrange's theorem.
theorem set_list_union_contains_head[K](head: Set[K], tail: List[Set[K]], x: K) { head.contains(x) implies set_list_union[K](List.cons[Set[K]](head, tail)).contains(x) } by {
    set_list_union[K](List.cons[Set[K]](head, tail)) = head.union(set_list_union[K](tail))
    head.union(set_list_union[K](tail)).contains(x)
}

/// An auxiliary proposition for Lagrange's theorem.
theorem set_list_union_contains_tail[K](head: Set[K], tail: List[Set[K]], x: K) { set_list_union[K](tail).contains(x) implies set_list_union[K](List.cons[Set[K]](head, tail)).contains(x) } by {
    set_list_union[K](List.cons[Set[K]](head, tail)) = head.union(set_list_union[K](tail))
    head.union(set_list_union[K](tail)).contains(x)
}

/// An auxiliary proposition for Lagrange's theorem.
theorem cons_contains_tail_of_contains_not_head[A](head: A, tail: List[A], item: A) { List.cons[A](head, tail).contains(item) and head != item implies tail.contains(item) } by {
    List.cons[A](head, tail).contains(item) = (if head = item { true } else { tail.contains(item) })
    if head = item {
        false
    }
    tail.contains(item)
}

/// An auxiliary proposition for Lagrange's theorem.
theorem set_list_union_contains_cons_step_head_eq[K](head: Set[K], tail: List[Set[K]], c: Set[K], x: K) { head = c and c.contains(x) implies set_list_union[K](List.cons[Set[K]](head, tail)).contains(x) } by {
    head.contains(x)
    set_list_union_contains_head[K](head, tail, x)
}

/// An auxiliary proposition for Lagrange's theorem.
theorem set_list_union_contains_cons_step_head_ne[K](head: Set[K], tail: List[Set[K]], c: Set[K], x: K) { head != c implies ((tail.contains(c) and c.contains(x) implies set_list_union[K](tail).contains(x)) implies (List.cons[Set[K]](head, tail).contains(c) and c.contains(x) implies set_list_union[K](List.cons[Set[K]](head, tail)).contains(x))) } by {
    if tail.contains(c) and c.contains(x) implies set_list_union[K](tail).contains(x) {
        if List.cons[Set[K]](head, tail).contains(c) and c.contains(x) {
            tail.contains(c)
            set_list_union[K](tail).contains(x)
            set_list_union[K](List.cons[Set[K]](head, tail)).contains(x)
        }
    }
}

/// An auxiliary proposition for Lagrange's theorem.
theorem set_list_union_contains_cons_step[K](head: Set[K], tail: List[Set[K]], c: Set[K], x: K) { (tail.contains(c) and c.contains(x) implies set_list_union[K](tail).contains(x)) implies (List.cons[Set[K]](head, tail).contains(c) and c.contains(x) implies set_list_union[K](List.cons[Set[K]](head, tail)).contains(x)) } by {
    if tail.contains(c) and c.contains(x) implies set_list_union[K](tail).contains(x) {
        if List.cons[Set[K]](head, tail).contains(c) and c.contains(x) {
            if head = c {
                set_list_union_contains_cons_step_head_eq[K](head, tail, c, x)
            } else {
                set_list_union_contains_cons_step_head_ne[K](head, tail, c, x)
            }
        }
    }
}

/// An auxiliary proposition for Lagrange's theorem.
theorem set_list_union_contains_cons_induction_step[K](head: Set[K], tail: List[Set[K]]) { (forall(d: Set[K], y: K) { tail.contains(d) and d.contains(y) implies set_list_union[K](tail).contains(y) }) implies (forall(d: Set[K], y: K) { List.cons[Set[K]](head, tail).contains(d) and d.contains(y) implies set_list_union[K](List.cons[Set[K]](head, tail)).contains(y) }) } by {
    forall(d: Set[K], y: K) {
        if List.cons[Set[K]](head, tail).contains(d) and d.contains(y) {
            tail.contains(d) and d.contains(y) implies set_list_union[K](tail).contains(y)
            set_list_union_contains_cons_step[K](head, tail, d, y)
            set_list_union[K](List.cons[Set[K]](head, tail)).contains(y)
        }
    }
}

// Proven helper: set_list_union_contains_of_member
/// An auxiliary proposition for Lagrange's theorem.
theorem set_list_union_contains_of_member[K](items: List[Set[K]], c: Set[K], x: K) {
    items.contains(c) and c.contains(x) implies set_list_union[K](items).contains(x)
} by {
    let p: List[Set[K]] -> Bool = function(xs: List[Set[K]]) {
        forall(d: Set[K], y: K) {
            xs.contains(d) and d.contains(y) implies set_list_union[K](xs).contains(y)
        }
    }
    forall(d: Set[K], y: K) {
        set_list_union_contains_nil_member[K](d, y)
    }
    p(List.nil[Set[K]])
    forall(head: Set[K], tail: List[Set[K]]) {
        if p(tail) {
            set_list_union_contains_cons_induction_step[K](head, tail)
            p(List.cons[Set[K]](head, tail))
        }
    }
    List.induction(p)
    forall(xs: List[Set[K]]) { p(xs) }
    p(items)
}

from nat import Nat

from data.basic.set import disjoint_union_is_length

/// The auxiliary set list all cardinality is object used in Lagrange's theorem.
define set_list_all_cardinality_is[K](items: List[Set[K]], n: Nat) -> Bool {
    forall(s: Set[K]) {
        items.contains(s) implies s.cardinality_is(n)
    }
}

// Proven helper: cons_contains_head

/// The auxiliary set list recursively disjoint object used in Lagrange's theorem.
define set_list_recursively_disjoint[K](items: List[Set[K]]) -> Bool {
    match items {
        List.nil {
            true
        }
        List.cons(head, tail) {
            head.is_disjoint(set_list_union[K](tail)) and set_list_recursively_disjoint[K](tail)
        }
    }
}

// Proven helper: set_list_recursively_disjoint_tail

/// An auxiliary proposition for Lagrange's theorem.
theorem cons_contains_head[A](head: A, tail: List[A]) { List.cons(head, tail).contains(head) }

/// An auxiliary proposition for Lagrange's theorem.
theorem set_list_recursively_disjoint_tail[K](head: Set[K], tail: List[Set[K]]) {
    set_list_recursively_disjoint[K](List.cons(head, tail)) implies set_list_recursively_disjoint[K](tail)
} by {
}

// Proven helper: set_list_recursively_disjoint_head

/// An auxiliary proposition for Lagrange's theorem.
theorem set_list_recursively_disjoint_head[K](head: Set[K], tail: List[Set[K]]) {
    set_list_recursively_disjoint[K](List.cons(head, tail)) implies head.is_disjoint(set_list_union[K](tail))
}

// Proven helper: mul_cons_length

/// An auxiliary proposition for Lagrange's theorem.
theorem mul_cons_length[A](head: A, tail: List[A], n: Nat) { n + n * tail.length = n * List.cons(head, tail).length }

// Proven helper: set_list_union_cardinality_recursive_cons_step

/// An auxiliary proposition for Lagrange's theorem.
theorem set_list_union_cardinality_recursive_cons_step[K](head: Set[K], tail: List[Set[K]], n: Nat) {
    head.cardinality_is(n) and
    set_list_union[K](tail).cardinality_is(n * tail.length) and
    set_list_recursively_disjoint[K](List.cons(head, tail))
    implies set_list_union[K](List.cons(head, tail)).cardinality_is(n * List.cons(head, tail).length)
} by {
    head.is_disjoint(set_list_union[K](tail))
    head.cardinality_is(n) and set_list_union[K](tail).cardinality_is(n * tail.length) and head.is_disjoint(set_list_union[K](tail))
    disjoint_union_is_length[K](head, set_list_union[K](tail), n, n * tail.length)
    head.union(set_list_union[K](tail)).cardinality_is(n + n * tail.length)
    mul_cons_length[Set[K]](head, tail, n)
    n + n * tail.length = n * List.cons(head, tail).length
    head.union(set_list_union[K](tail)).cardinality_is(n * List.cons(head, tail).length)
    set_list_union[K](List.cons(head, tail)) = head.union(set_list_union[K](tail))
}

// Proven helper: set_list_union_cardinality_recursive_nil

/// An auxiliary proposition for Lagrange's theorem.
theorem set_list_union_cardinality_recursive_nil[K](n: Nat) {
    set_list_all_cardinality_is[K](List.nil[Set[K]], n) and
    set_list_recursively_disjoint[K](List.nil[Set[K]])
    implies set_list_union[K](List.nil[Set[K]]).cardinality_is(n * List.nil[Set[K]].length)
} by {
    Set.empty_set[K].cardinality_is(Nat.0)
    set_list_union[K](List.nil[Set[K]]) = Set.empty_set[K]
    List.nil[Set[K]].length = Nat.0
    n * List.nil[Set[K]].length = Nat.0
}

/// An auxiliary proposition for Lagrange's theorem.
theorem set_list_union_nil_cardinality_is_zero[K] {
    set_list_union[K](List.nil[Set[K]]).cardinality_is(Nat.0)
} by {
    set_list_union[K](List.nil[Set[K]]) = Set[K].empty_set
    Set[K].empty_set.cardinality_is(Nat.0)
}

// Proven helper: list_forall_contains_elim

/// An auxiliary proposition for Lagrange's theorem.
theorem list_forall_contains_elim[A](items: List[A], p: A -> Bool, s: A) { (forall(x: A) { items.contains(x) implies p(x) }) and items.contains(s) implies p(s) }

// Proven helper: set_list_all_cardinality_is_elim

/// An auxiliary proposition for Lagrange's theorem.
theorem set_list_all_cardinality_is_elim[K](items: List[Set[K]], n: Nat, s: Set[K]) {
    set_list_all_cardinality_is[K](items, n) and items.contains(s) implies s.cardinality_is(n)
} by {
    define p(x: Set[K]) -> Bool { x.cardinality_is(n) }
    list_forall_contains_elim[Set[K]](items, p, s)
    p(s)
    s.cardinality_is(n)
}

// Proven helper: cons_contains_tail

/// An auxiliary proposition for Lagrange's theorem.
theorem cons_contains_tail[A](head: A, tail: List[A], item: A) {
    tail.contains(item) implies List.cons(head, tail).contains(item)
}

// Proven helper: set_list_all_cardinality_tail_point

/// An auxiliary proposition for Lagrange's theorem.
theorem set_list_all_cardinality_tail_point[K](head: Set[K], tail: List[Set[K]], n: Nat, s: Set[K]) {
    set_list_all_cardinality_is[K](List.cons(head, tail), n) and tail.contains(s) implies s.cardinality_is(n)
} by {
    cons_contains_tail[Set[K]](head, tail, s)
    List.cons(head, tail).contains(s)
    set_list_all_cardinality_is_elim[K](List.cons(head, tail), n, s)
}

// Proven helper: set_list_all_cardinality_tail

/// An auxiliary proposition for Lagrange's theorem.
theorem set_list_all_cardinality_tail[K](head: Set[K], tail: List[Set[K]], n: Nat) {
    set_list_all_cardinality_is[K](List.cons(head, tail), n) implies
    set_list_all_cardinality_is[K](tail, n)
} by {
    forall(s: Set[K]) {
        if tail.contains(s) {
            set_list_all_cardinality_tail_point[K](head, tail, n, s)
            s.cardinality_is(n)
        }
    }
}

// Proven helper: set_list_all_cardinality_head

/// An auxiliary proposition for Lagrange's theorem.
theorem set_list_all_cardinality_head[K](head: Set[K], tail: List[Set[K]], n: Nat) {
    set_list_all_cardinality_is[K](List.cons(head, tail), n) implies head.cardinality_is(n)
} by {
    cons_contains_head[Set[K]](head, tail)
    set_list_all_cardinality_is_elim[K](List.cons(head, tail), n, head)
}

// Proven helper: set_list_union_cardinality_recursive

/// An auxiliary proposition for Lagrange's theorem.
theorem set_list_union_cardinality_recursive[K](items: List[Set[K]], n: Nat) {
    set_list_all_cardinality_is[K](items, n) and
    set_list_recursively_disjoint[K](items)
    implies set_list_union[K](items).cardinality_is(n * items.length)
} by {
    define p(xs: List[Set[K]]) -> Bool {
        set_list_all_cardinality_is[K](xs, n) and
        set_list_recursively_disjoint[K](xs)
        implies set_list_union[K](xs).cardinality_is(n * xs.length)
    }
    set_list_union_cardinality_recursive_nil[K](n)
    p(List.nil[Set[K]])
    forall(head: Set[K], tail: List[Set[K]]) {
        if p(tail) {
            if set_list_all_cardinality_is[K](List.cons(head, tail), n) and set_list_recursively_disjoint[K](List.cons(head, tail)) {
                set_list_all_cardinality_head[K](head, tail, n)
                head.cardinality_is(n)
                set_list_all_cardinality_tail[K](head, tail, n)
                set_list_all_cardinality_is[K](tail, n)
                set_list_recursively_disjoint_tail[K](head, tail)
                set_list_recursively_disjoint[K](tail)
                set_list_union[K](tail).cardinality_is(n * tail.length)
                set_list_union_cardinality_recursive_cons_step[K](head, tail, n)
                set_list_union[K](List.cons(head, tail)).cardinality_is(n * List.cons(head, tail).length)
            }
            p(List.cons(head, tail))
        }
    }
    List.induction(function(xs: List[Set[K]]) { p(xs) })
    forall(xs: List[Set[K]]) { p(xs) }
    p(items)
}

from list import unique_implies_tail_unique, unique_implies_no_duplicate, list_contains_implies_count_geq_one

from nat import sum_lte

from data.basic.set import union_contains_cases

/// The auxiliary set list pairwise disjoint object used in Lagrange's theorem.
define set_list_pairwise_disjoint[K](items: List[Set[K]]) -> Bool {
    forall(a: Set[K], b: Set[K]) {
        items.contains(a) and items.contains(b) and a != b implies a.is_disjoint(b)
    }
}

/// An auxiliary proposition for Lagrange's theorem.
theorem cons_unique_not_tail_contains_light[A](head: A, tail: List[A]) {
    List.cons(head, tail).is_unique implies not tail.contains(head)
} by {
    if tail.contains(head) {
        list_contains_implies_count_geq_one[A](tail, head)
        tail.count(head) >= Nat.1
        List.cons(head, tail).count(head) = Nat.1 + tail.count(head)
        sum_lte(Nat.1, Nat.1, Nat.1, tail.count(head))
        Nat.1 + Nat.1 <= Nat.1 + tail.count(head)
        Nat.1 + Nat.1 = Nat.1.suc
        Nat.1.suc <= Nat.1 + tail.count(head)
        List.cons(head, tail).count(head) >= Nat.1.suc
        List.cons(head, tail).count(head) > Nat.1
        unique_implies_no_duplicate[A](List.cons(head, tail), head)
        List.cons(head, tail).count(head) <= Nat.1
        false
    }
}

/// An auxiliary proposition for Lagrange's theorem.
theorem set_is_disjoint_union_of_disjoint[K](a: Set[K], b: Set[K], c: Set[K]) {
    a.is_disjoint(b) and a.is_disjoint(c) implies a.is_disjoint(b.union(c))
} by {
    forall(x: K) {
        if a.contains(x) and b.union(c).contains(x) {
            union_contains_cases(b, c, x)
            if b.contains(x) {
                a.contains(x) and b.contains(x)
                false
            }
            if c.contains(x) {
                a.contains(x) and c.contains(x)
                false
            }
            false
        }
    }
}

/// An auxiliary proposition for Lagrange's theorem.
theorem set_disjoint_from_all_implies_disjoint_union_nil[K](s: Set[K]) {
    (forall(t: Set[K]) { List.nil[Set[K]].contains(t) implies s.is_disjoint(t) }) implies s.is_disjoint(set_list_union[K](List.nil[Set[K]]))
} by {
    set_list_union[K](List.nil[Set[K]]) = Set.empty_set[K]
    forall(x: K) {
        not (s.contains(x) and Set.empty_set[K].contains(x))
    }
    s.is_disjoint(Set.empty_set[K])
    s.is_disjoint(set_list_union[K](List.nil[Set[K]]))
}

/// The auxiliary set disjoint from all union statement object used in Lagrange's theorem.
define set_disjoint_from_all_union_statement[K](s: Set[K], items: List[Set[K]]) -> Bool {
    (forall(t: Set[K]) { items.contains(t) implies s.is_disjoint(t) }) implies s.is_disjoint(set_list_union[K](items))
}

/// An auxiliary proposition for Lagrange's theorem.
theorem set_disjoint_from_all_union_statement_nil[K](s: Set[K]) {
    set_disjoint_from_all_union_statement[K](s, List.nil[Set[K]])
} by {
    set_disjoint_from_all_implies_disjoint_union_nil[K](s)
}

/// An auxiliary proposition for Lagrange's theorem.
theorem set_disjoint_from_all_union_statement_cons_step[K](s: Set[K], head: Set[K], tail: List[Set[K]]) {
    set_disjoint_from_all_union_statement[K](s, tail) implies set_disjoint_from_all_union_statement[K](s, List.cons(head, tail))
} by {
    if set_disjoint_from_all_union_statement[K](s, tail) {
        set_disjoint_from_all_union_statement[K](s, tail) = ((forall(t: Set[K]) { tail.contains(t) implies s.is_disjoint(t) }) implies s.is_disjoint(set_list_union[K](tail)))
        if forall(t: Set[K]) { List.cons(head, tail).contains(t) implies s.is_disjoint(t) } {
            List.cons(head, tail).contains(head)
            s.is_disjoint(head)
            forall(t: Set[K]) {
                if tail.contains(t) {
                    List.cons(head, tail).contains(t)
                    s.is_disjoint(t)
                }
            }
            forall(t: Set[K]) { tail.contains(t) implies s.is_disjoint(t) }
            s.is_disjoint(set_list_union[K](tail))
            set_is_disjoint_union_of_disjoint[K](s, head, set_list_union[K](tail))
            s.is_disjoint(head.union(set_list_union[K](tail)))
            set_list_union[K](List.cons(head, tail)) = head.union(set_list_union[K](tail))
            s.is_disjoint(set_list_union[K](List.cons(head, tail)))
        }
    }
}

/// An auxiliary proposition for Lagrange's theorem.
theorem set_disjoint_from_all_implies_disjoint_union[K](s: Set[K], items: List[Set[K]]) {
    (forall(t: Set[K]) { items.contains(t) implies s.is_disjoint(t) }) implies s.is_disjoint(set_list_union[K](items))
} by {
    let p: List[Set[K]] -> Bool = function(xs: List[Set[K]]) {
        (forall(t: Set[K]) { xs.contains(t) implies s.is_disjoint(t) }) implies s.is_disjoint(set_list_union[K](xs))
    }
    set_disjoint_from_all_implies_disjoint_union_nil[K](s)
    p(List.nil[Set[K]])
    forall(tail: List[Set[K]], head: Set[K]) {
        if p(tail) {
            p(tail) = ((forall(t: Set[K]) { tail.contains(t) implies s.is_disjoint(t) }) implies s.is_disjoint(set_list_union[K](tail)))
            if forall(t: Set[K]) { List.cons(head, tail).contains(t) implies s.is_disjoint(t) } {
                List.cons(head, tail).contains(head)
                s.is_disjoint(head)
                forall(t: Set[K]) {
                    if tail.contains(t) {
                        List.cons(head, tail).contains(t)
                        s.is_disjoint(t)
                    }
                }
                forall(t: Set[K]) { tail.contains(t) implies s.is_disjoint(t) }
                s.is_disjoint(set_list_union[K](tail))
                set_is_disjoint_union_of_disjoint[K](s, head, set_list_union[K](tail))
                s.is_disjoint(head.union(set_list_union[K](tail)))
                set_list_union[K](List.cons(head, tail)) = head.union(set_list_union[K](tail))
                s.is_disjoint(set_list_union[K](List.cons(head, tail)))
            }
            p(List.cons(head, tail))
        }
    }
    List.induction(p)
    p(items)
    p(items) = ((forall(t: Set[K]) { items.contains(t) implies s.is_disjoint(t) }) implies s.is_disjoint(set_list_union[K](items)))
}

/// An auxiliary proposition for Lagrange's theorem.
theorem set_list_pairwise_disjoint_elim_curried[K](items: List[Set[K]], a: Set[K], b: Set[K]) {
    set_list_pairwise_disjoint[K](items) implies (items.contains(a) implies (items.contains(b) implies (a != b implies a.is_disjoint(b))))
} by {
    set_list_pairwise_disjoint[K](items) = forall(x: Set[K], y: Set[K]) { items.contains(x) and items.contains(y) and x != y implies x.is_disjoint(y) }
    forall(x: Set[K], y: Set[K]) { items.contains(x) and items.contains(y) and x != y implies x.is_disjoint(y) }
    if items.contains(a) {
        if items.contains(b) {
            if a != b {
                items.contains(a) and items.contains(b) and a != b
                a.is_disjoint(b)
            }
        }
    }
}

/// An auxiliary proposition for Lagrange's theorem.
theorem set_list_pairwise_disjoint_tail_point[K](head: Set[K], tail: List[Set[K]], a: Set[K], b: Set[K]) {
    set_list_pairwise_disjoint[K](List.cons(head, tail)) and tail.contains(a) and tail.contains(b) and a != b implies a.is_disjoint(b)
} by {
    cons_contains_tail[Set[K]](head, tail, a)
    cons_contains_tail[Set[K]](head, tail, b)
    List.cons(head, tail).contains(a)
    List.cons(head, tail).contains(b)
    set_list_pairwise_disjoint_elim_curried[K](List.cons(head, tail), a, b)
}

/// An auxiliary proposition for Lagrange's theorem.
theorem set_list_pairwise_disjoint_tail[K](head: Set[K], tail: List[Set[K]]) {
    set_list_pairwise_disjoint[K](List.cons(head, tail)) implies set_list_pairwise_disjoint[K](tail)
} by {
    forall(a: Set[K], b: Set[K]) {
        if tail.contains(a) and tail.contains(b) and a != b {
            set_list_pairwise_disjoint_tail_point[K](head, tail, a, b)
            a.is_disjoint(b)
        }
    }
}

/// An auxiliary proposition for Lagrange's theorem.
theorem set_list_pairwise_disjoint_cons_head_tail_point[K](head: Set[K], tail: List[Set[K]], s: Set[K]) {
    set_list_pairwise_disjoint[K](List.cons(head, tail)) and not tail.contains(head) and tail.contains(s) implies head.is_disjoint(s)
} by {
    cons_contains_head[Set[K]](head, tail)
    cons_contains_tail[Set[K]](head, tail, s)
    List.cons(head, tail).contains(head)
    List.cons(head, tail).contains(s)
    if head = s {
        tail.contains(head)
        false
    }
    head != s
    set_list_pairwise_disjoint_elim_curried[K](List.cons(head, tail), head, s)
}

/// An auxiliary proposition for Lagrange's theorem.
theorem set_list_pairwise_disjoint_cons_head_tail_all[K](head: Set[K], tail: List[Set[K]]) {
    set_list_pairwise_disjoint[K](List.cons(head, tail)) and not tail.contains(head) implies forall(s: Set[K]) { tail.contains(s) implies head.is_disjoint(s) }
} by {
    forall(s: Set[K]) {
        if tail.contains(s) {
            set_list_pairwise_disjoint_cons_head_tail_point[K](head, tail, s)
            head.is_disjoint(s)
        }
    }
}

/// An auxiliary proposition for Lagrange's theorem.
theorem unique_pairwise_implies_recursively_disjoint_nil[K] {
    List.nil[Set[K]].is_unique and set_list_pairwise_disjoint[K](List.nil[Set[K]]) implies set_list_recursively_disjoint[K](List.nil[Set[K]])
} by {
    set_list_recursively_disjoint[K](List.nil[Set[K]]) = true
}

/// An auxiliary proposition for Lagrange's theorem.
theorem unique_pairwise_implies_recursively_disjoint_cons_step[K](head: Set[K], tail: List[Set[K]]) {
    (tail.is_unique and set_list_pairwise_disjoint[K](tail) implies set_list_recursively_disjoint[K](tail)) and
    List.cons(head, tail).is_unique and set_list_pairwise_disjoint[K](List.cons(head, tail))
    implies set_list_recursively_disjoint[K](List.cons(head, tail))
} by {
    unique_implies_tail_unique[Set[K]](head, tail)
    tail.is_unique
    set_list_pairwise_disjoint_tail[K](head, tail)
    set_list_pairwise_disjoint[K](tail)
    set_list_recursively_disjoint[K](tail)
    cons_unique_not_tail_contains_light[Set[K]](head, tail)
    not tail.contains(head)
    set_list_pairwise_disjoint_cons_head_tail_all[K](head, tail)
    forall(s: Set[K]) { tail.contains(s) implies head.is_disjoint(s) }
    set_disjoint_from_all_implies_disjoint_union[K](head, tail)
    head.is_disjoint(set_list_union[K](tail))
    head.is_disjoint(set_list_union[K](tail)) and set_list_recursively_disjoint[K](tail)
    set_list_recursively_disjoint[K](List.cons(head, tail)) = (head.is_disjoint(set_list_union[K](tail)) and set_list_recursively_disjoint[K](tail))
    set_list_recursively_disjoint[K](List.cons(head, tail))
}

/// The auxiliary unique pairwise recursive statement object used in Lagrange's theorem.
define unique_pairwise_recursive_statement[K](items: List[Set[K]]) -> Bool {
    items.is_unique and set_list_pairwise_disjoint[K](items) implies set_list_recursively_disjoint[K](items)
}

/// An auxiliary proposition for Lagrange's theorem.
theorem unique_pairwise_recursive_statement_nil[K] {
    unique_pairwise_recursive_statement[K](List.nil[Set[K]])
} by {
    unique_pairwise_implies_recursively_disjoint_nil[K]
}

/// An auxiliary proposition for Lagrange's theorem.
theorem unique_pairwise_recursive_statement_cons_step[K](head: Set[K], tail: List[Set[K]]) {
    unique_pairwise_recursive_statement[K](tail) implies unique_pairwise_recursive_statement[K](List.cons(head, tail))
} by {
    if unique_pairwise_recursive_statement[K](tail) {
        unique_pairwise_recursive_statement[K](tail) = (tail.is_unique and set_list_pairwise_disjoint[K](tail) implies set_list_recursively_disjoint[K](tail))
        unique_pairwise_implies_recursively_disjoint_cons_step[K](head, tail)
    }
}

/// An auxiliary proposition for Lagrange's theorem.
theorem unique_pairwise_implies_recursively_disjoint[K](items: List[Set[K]]) {
    items.is_unique and set_list_pairwise_disjoint[K](items)
    implies set_list_recursively_disjoint[K](items)
} by {
    let p: List[Set[K]] -> Bool = function(xs: List[Set[K]]) {
        xs.is_unique and set_list_pairwise_disjoint[K](xs) implies set_list_recursively_disjoint[K](xs)
    }
    unique_pairwise_recursive_statement_nil[K]
    p(List.nil[Set[K]])
    forall(tail: List[Set[K]], head: Set[K]) {
        if p(tail) {
            p(tail) = (tail.is_unique and set_list_pairwise_disjoint[K](tail) implies set_list_recursively_disjoint[K](tail))
            unique_pairwise_recursive_statement_cons_step[K](head, tail)
            p(List.cons(head, tail))
        }
    }
    List.induction(p)
    p(items)
    p(items) = (items.is_unique and set_list_pairwise_disjoint[K](items) implies set_list_recursively_disjoint[K](items))
}

from algebra.group import Group, left_cancel
from algebra.subgroup import Subgroup, subgroup_contains_identity
from data.basic.functions import is_injective_fn
from finite_group import FiniteGroup, FiniteSubgroup, finite_subgroup_as_set_cardinality_is_order, finite_subgroup_as_set_contains_finite_subgroup_eq, finite_subgroup_contains_as_subgroup_eq
from data.basic.set import set_image, set_image_cardinality_is_of_injective, set_image_contains_witness, image_contains, set_image_contains_eq, set_ext, set_eq_transport_predicate, list_indexed_union, list_indexed_union_contains_of_contains, set_eq_universal_of_forall_contains
from list import map, unique_list_is_unique, unique_preserves_contains, unique_contains_imp_contains
from list import map_contains

/// An auxiliary proposition for Lagrange's theorem.
theorem same_left_coset_raw_reflexive[G: Group](s: Subgroup[G], a: G) { s.contains(a.inverse * a) } by {
    a.inverse * a = G.1
    subgroup_contains_identity[G](s)
}

// AixBot-proven seed helper from 2026-05-06 #71 run.

/// An auxiliary proposition for Lagrange's theorem.
theorem same_left_coset_raw_symmetric[G: Group](s: Subgroup[G], a: G, b: G) { s.contains(a.inverse * b) implies s.contains(b.inverse * a) } by {
    s.contains((a.inverse * b).inverse)
    (a.inverse * b).inverse = b.inverse * a
    s.contains(b.inverse * a)
}

// AixBot-proven seed helper from 2026-05-06 #71 run.

/// An auxiliary proposition for Lagrange's theorem.
theorem same_left_coset_raw_transitive[G: Group](s: Subgroup[G], a: G, b: G, c: G) { s.contains(a.inverse * b) and s.contains(b.inverse * c) implies s.contains(a.inverse * c) } by {
    s.contains((a.inverse * b) * (b.inverse * c))
    (a.inverse * b) * (b.inverse * c) = a.inverse * c
    s.contains(a.inverse * c)
}

// AixBot-proven seed helper from 2026-05-06 #71 run.

/// An auxiliary proposition for Lagrange's theorem.
theorem left_coset_intersection_witness_implies_same_representative_raw[G: Group](s: Subgroup[G], a: G, b: G, x: G) { s.contains(a.inverse * x) and s.contains(b.inverse * x) implies s.contains(a.inverse * b) } by {
    same_left_coset_raw_symmetric[G](s, b, x)
    s.contains(x.inverse * b)
    same_left_coset_raw_transitive[G](s, a, x, b)
}

// AixBot-proven seed helper from 2026-05-06 #71 run.

/// An auxiliary proposition for Lagrange's theorem.
theorem same_representative_implies_left_coset_membership_eq_raw[G: Group](s: Subgroup[G], a: G, b: G, y: G) { s.contains(a.inverse * b) implies s.contains(a.inverse * y) = s.contains(b.inverse * y) } by {
    if s.contains(a.inverse * y) {
        same_left_coset_raw_symmetric[G](s, a, b)
        s.contains(b.inverse * a)
        same_left_coset_raw_transitive[G](s, b, a, y)
        s.contains(b.inverse * y)
    }
    if s.contains(b.inverse * y) {
        same_left_coset_raw_transitive[G](s, a, b, y)
        s.contains(a.inverse * y)
    }
}

// AixBot-proven seed helper from 2026-05-06 #71 run.

/// The auxiliary finite left coset set object used in Lagrange's theorem.
define finite_left_coset_set[G: FiniteGroup](s: FiniteSubgroup[G], a: G) -> Set[G] {
    Set[G].new(function(y: G) { s.contains(a.inverse * y) })
}

// Proven helper: finite_left_coset_set_contains_eq_local

/// An auxiliary proposition for Lagrange's theorem.
theorem finite_left_coset_set_contains_eq_local[G: FiniteGroup](s: FiniteSubgroup[G], a: G, y: G) {
    finite_left_coset_set[G](s, a).contains(y) = s.contains(a.inverse * y)
}

/// An auxiliary proposition for Lagrange's theorem.
theorem same_representative_left_coset_membership_eq_clause[G: Group](s: Subgroup[G], a: G, b: G, y: G) { s.contains(a.inverse * b) implies s.contains(a.inverse * y) = s.contains(b.inverse * y) } by {
    same_representative_implies_left_coset_membership_eq_raw[G](s, a, b, y)
}

// Proven helper: left_mul_is_injective_fn

/// An auxiliary proposition for Lagrange's theorem.
theorem left_mul_is_injective_fn[G: Group](a: G) {
    is_injective_fn[G, G](function(h: G) { a * h })
} by {
    forall(x: G, y: G) {
        if (function(h: G) { a * h })(x) = (function(h: G) { a * h })(y) {
            left_cancel[G](a, x, y)
        }
    }
}

// Proven helper: finite_subgroup_left_mul_image_cardinality_is_order

/// An auxiliary proposition for Lagrange's theorem.
theorem finite_subgroup_left_mul_image_cardinality_is_order[G: FiniteGroup](s: FiniteSubgroup[G], a: G) {
    set_image[G, G](s.as_set, function(h: G) { a * h }).cardinality_is(s.order)
} by {
    finite_subgroup_as_set_cardinality_is_order[G](s)
    left_mul_is_injective_fn[G](a)
    set_image_cardinality_is_of_injective[G, G](s.as_set, function(h: G) { a * h }, s.order)
}

/// An auxiliary proposition for Lagrange's theorem.
theorem left_coset_intersection_witness_same_representative_clause[G: Group](s: Subgroup[G], a: G, b: G, x: G) { s.contains(a.inverse * x) and s.contains(b.inverse * x) implies s.contains(a.inverse * b) } by {
    left_coset_intersection_witness_implies_same_representative_raw[G](s, a, b, x)
}

// Proven helper: same_representative_left_coset_membership_eq_clause

/// An auxiliary proposition for Lagrange's theorem.
theorem left_mul_subgroup_left_coset_injective_raw[G: Group](s: Subgroup[G], a: G, h1: G, h2: G) { s.contains(h1) and s.contains(h2) and a * h1 = a * h2 implies h1 = h2 } by {
    h1 = h2
}

// Proven helper: left_coset_representative_mem_raw_clause

/// An auxiliary proposition for Lagrange's theorem.
theorem left_coset_representative_mem_raw_clause[G: Group](s: Subgroup[G], a: G) { s.contains(a.inverse * a) } by {
    same_left_coset_raw_reflexive[G](s, a)
}

// Proven helper: left_mul_subgroup_image_mem_clause

/// An auxiliary proposition for Lagrange's theorem.
theorem left_mul_subgroup_image_mem_clause[G: Group](s: Subgroup[G], a: G, h: G) { s.contains(h) implies s.contains(a.inverse * (a * h)) = true } by {
    s.contains(a.inverse * (a * h))
}

// Proven helper: left_mul_subgroup_left_coset_injective_clause

/// An auxiliary proposition for Lagrange's theorem.
theorem left_mul_subgroup_left_coset_injective_clause[G: Group](s: Subgroup[G], a: G, h1: G, h2: G) { s.contains(h1) and s.contains(h2) and a * h1 = a * h2 implies h1 = h2 } by {
    left_mul_subgroup_left_coset_injective_raw[G](s, a, h1, h2)
}

// Proven helper: left_coset_intersection_witness_same_representative_clause

/// An auxiliary proposition for Lagrange's theorem.
theorem left_coset_elem_has_subgroup_preimage_raw[G: Group](s: Subgroup[G], a: G, y: G) { s.contains(a.inverse * y) implies exists(h: G) { s.contains(h) and a * h = y } } by {
    a * (a.inverse * y) = (a * a.inverse) * y
    a * a.inverse = G.1
    (a * a.inverse) * y = G.1 * y
    G.1 * y = y
    a * (a.inverse * y) = y
    exists(h: G) { h = a.inverse * y and s.contains(h) and a * h = y }
    exists(h: G) { s.contains(h) and a * h = y }
}

// Proven helper: left_mul_subgroup_left_coset_injective_raw

// Proven helper: left_coset_elem_has_subgroup_preimage_clause

/// An auxiliary proposition for Lagrange's theorem.
theorem left_coset_elem_has_subgroup_preimage_clause[G: Group](s: Subgroup[G], a: G, y: G) { s.contains(a.inverse * y) implies exists(h: G) { s.contains(h) and a * h = y } } by {
    left_coset_elem_has_subgroup_preimage_raw[G](s, a, y)
}

/// An auxiliary proposition for Lagrange's theorem.
theorem left_mul_subgroup_into_left_coset_raw[G: Group](s: Subgroup[G], a: G, h: G) { s.contains(h) implies s.contains(a.inverse * (a * h)) } by {
    a.inverse * (a * h) = (a.inverse * a) * h
    a.inverse * a = G.1
    (a.inverse * a) * h = G.1 * h
    G.1 * h = h
    a.inverse * (a * h) = h
    s.contains(a.inverse * (a * h)) = s.contains(h)
    s.contains(a.inverse * (a * h))
}

// Proven helper: left_coset_elem_has_subgroup_preimage_raw

// Proven helper: left_mul_subgroup_into_left_coset_clause

/// An auxiliary proposition for Lagrange's theorem.
theorem left_mul_subgroup_into_left_coset_clause[G: Group](s: Subgroup[G], a: G, h: G) { s.contains(h) implies s.contains(a.inverse * (a * h)) } by {
    left_mul_subgroup_into_left_coset_raw[G](s, a, h)
}

/// An auxiliary proposition for Lagrange's theorem.
theorem intersecting_left_cosets_equal_raw[G: Group](s: Subgroup[G], a: G, b: G) { exists(x: G) { s.contains(a.inverse * x) and s.contains(b.inverse * x) } implies forall(y: G) { s.contains(a.inverse * y) = s.contains(b.inverse * y) } } by {
    let x: G satisfy { s.contains(a.inverse * x) and s.contains(b.inverse * x) }
    left_coset_intersection_witness_implies_same_representative_raw[G](s, a, b, x)
    s.contains(a.inverse * b)
    forall(y: G) {
        same_representative_implies_left_coset_membership_eq_raw[G](s, a, b, y)
        s.contains(a.inverse * y) = s.contains(b.inverse * y)
    }
}

// Proven helper: left_cosets_equal_or_disjoint_raw

/// An auxiliary proposition for Lagrange's theorem.
theorem left_cosets_equal_or_disjoint_raw[G: Group](s: Subgroup[G], a: G, b: G) { (forall(y: G) { s.contains(a.inverse * y) = s.contains(b.inverse * y) }) or not(exists(x: G) { s.contains(a.inverse * x) and s.contains(b.inverse * x) }) } by {
    if exists(x: G) { s.contains(a.inverse * x) and s.contains(b.inverse * x) } {
        intersecting_left_cosets_equal_raw[G](s, a, b)
        forall(y: G) { s.contains(a.inverse * y) = s.contains(b.inverse * y) }
    }
}

// Proven helper: left_mul_subgroup_into_left_coset_raw

// Proven helper: intersecting_left_cosets_pointwise_equal_clause

/// An auxiliary proposition for Lagrange's theorem.
theorem intersecting_left_cosets_pointwise_equal_clause[G: Group](s: Subgroup[G], a: G, b: G) { exists(x: G) { s.contains(a.inverse * x) and s.contains(b.inverse * x) } implies forall(y: G) { s.contains(a.inverse * y) = s.contains(b.inverse * y) } } by {
    intersecting_left_cosets_equal_raw[G](s, a, b)
}

// Proven helper: left_cosets_equal_or_disjoint_raw_clause

/// An auxiliary proposition for Lagrange's theorem.
theorem left_cosets_equal_or_disjoint_raw_clause[G: Group](s: Subgroup[G], a: G, b: G) { (forall(y: G) { s.contains(a.inverse * y) = s.contains(b.inverse * y) }) or not(exists(x: G) { s.contains(a.inverse * x) and s.contains(b.inverse * x) }) } by {
    left_cosets_equal_or_disjoint_raw[G](s, a, b)
}

// Proven helper: finite_left_mul_image_member_implies_left_coset_set_member

/// An auxiliary proposition for Lagrange's theorem.
theorem finite_left_mul_image_member_implies_left_coset_set_member[G: FiniteGroup](s: FiniteSubgroup[G], a: G, y: G) {
    set_image[G, G](s.as_set, function(h: G) { a * h }).contains(y) implies finite_left_coset_set[G](s, a).contains(y)
} by {
    set_image_contains_witness[G, G](s.as_set, function(h: G) { a * h }, y)
    let h: G satisfy { s.as_set.contains(h) and y = a * h }
    finite_subgroup_as_set_contains_finite_subgroup_eq[G](s, h)
    s.contains(h)
    finite_subgroup_contains_as_subgroup_eq[G](s, h)
    s.as_subgroup.contains(h)
    left_mul_subgroup_into_left_coset_clause[G](s.as_subgroup, a, h)
    s.as_subgroup.contains(a.inverse * (a * h))
    a.inverse * y = a.inverse * (a * h)
    s.as_subgroup.contains(a.inverse * y)
    finite_subgroup_contains_as_subgroup_eq[G](s, a.inverse * y)
    s.contains(a.inverse * y)
    finite_left_coset_set[G](s, a).contains(y)
}

// Proven helper: finite_left_coset_set_member_implies_left_mul_image_member

/// An auxiliary proposition for Lagrange's theorem.
theorem finite_left_coset_set_member_implies_left_mul_image_member[G: FiniteGroup](s: FiniteSubgroup[G], a: G, y: G) {
    finite_left_coset_set[G](s, a).contains(y) implies set_image[G, G](s.as_set, function(h: G) { a * h }).contains(y)
} by {
    finite_left_coset_set_contains_eq_local[G](s, a, y)
    s.contains(a.inverse * y)
    finite_subgroup_contains_as_subgroup_eq[G](s, a.inverse * y)
    s.as_subgroup.contains(a.inverse * y)
    left_coset_elem_has_subgroup_preimage_clause[G](s.as_subgroup, a, y)
    let h: G satisfy { s.as_subgroup.contains(h) and a * h = y }
    s.as_subgroup.contains(h)
    finite_subgroup_contains_as_subgroup_eq[G](s, h)
    s.contains(h)
    finite_subgroup_as_set_contains_finite_subgroup_eq[G](s, h)
    s.as_set.contains(h)
    y = a * h
    image_contains[G, G](function(k: G) { a * k }, s.as_set, y)
    set_image_contains_eq[G, G](s.as_set, function(k: G) { a * k }, y)
    set_image[G, G](s.as_set, function(k: G) { a * k }).contains(y)
}

// Proven helper: finite_left_mul_image_contains_eq_left_coset_set_contains

/// An auxiliary proposition for Lagrange's theorem.
theorem finite_left_mul_image_contains_eq_left_coset_set_contains[G: FiniteGroup](s: FiniteSubgroup[G], a: G, y: G) {
    set_image[G, G](s.as_set, function(h: G) { a * h }).contains(y) = finite_left_coset_set[G](s, a).contains(y)
} by {
    if set_image[G, G](s.as_set, function(h: G) { a * h }).contains(y) {
        finite_left_mul_image_member_implies_left_coset_set_member[G](s, a, y)
    }
    if finite_left_coset_set[G](s, a).contains(y) {
        finite_left_coset_set_member_implies_left_mul_image_member[G](s, a, y)
    }
}

// Proven helper: finite_subgroup_left_mul_image_eq_left_coset_set

/// An auxiliary proposition for Lagrange's theorem.
theorem finite_subgroup_left_mul_image_eq_left_coset_set[G: FiniteGroup](s: FiniteSubgroup[G], a: G) {
    set_image[G, G](s.as_set, function(h: G) { a * h }) = finite_left_coset_set[G](s, a)
} by {
    forall(y: G) {
        finite_left_mul_image_contains_eq_left_coset_set_contains[G](s, a, y)
    }
    set_ext[G](set_image[G, G](s.as_set, function(h: G) { a * h }), finite_left_coset_set[G](s, a))
}

// Proven helper: finite_left_coset_set_cardinality_is_order

/// An auxiliary proposition for Lagrange's theorem.
theorem finite_left_coset_set_cardinality_is_order[G: FiniteGroup](s: FiniteSubgroup[G], a: G) {
    finite_left_coset_set[G](s, a).cardinality_is(s.order)
} by {
    let image: Set[G] = set_image[G, G](s.as_set, function(h: G) { a * h })
    let coset: Set[G] = finite_left_coset_set[G](s, a)
    let p: Set[G] -> Bool = function(t: Set[G]) { t.cardinality_is(s.order) }
    finite_subgroup_left_mul_image_cardinality_is_order[G](s, a)
    finite_subgroup_left_mul_image_eq_left_coset_set[G](s, a)
    set_eq_transport_predicate[G](p, image, coset)
}

// Proven helper: finite_left_coset_sets_pointwise_eq_from_raw

/// An auxiliary proposition for Lagrange's theorem.
theorem finite_left_coset_sets_pointwise_eq_from_raw[G: FiniteGroup](s: FiniteSubgroup[G], a: G, b: G) {
    (forall(y: G) { s.as_subgroup.contains(a.inverse * y) = s.as_subgroup.contains(b.inverse * y) }) implies
    forall(y: G) { finite_left_coset_set[G](s, a).contains(y) = finite_left_coset_set[G](s, b).contains(y) }
} by {
    forall(y: G) {
        finite_left_coset_set[G](s, a).contains(y) = s.contains(a.inverse * y)
        s.contains(a.inverse * y) = s.as_subgroup.contains(a.inverse * y)
        s.as_subgroup.contains(a.inverse * y) = s.as_subgroup.contains(b.inverse * y)
        s.as_subgroup.contains(b.inverse * y) = s.contains(b.inverse * y)
        s.contains(b.inverse * y) = finite_left_coset_set[G](s, b).contains(y)
        finite_left_coset_set[G](s, a).contains(y) = finite_left_coset_set[G](s, b).contains(y)
    }
}

// Proven helper: finite_left_coset_sets_eq_from_raw_pointwise

/// An auxiliary proposition for Lagrange's theorem.
theorem finite_left_coset_sets_eq_from_raw_pointwise[G: FiniteGroup](s: FiniteSubgroup[G], a: G, b: G) {
    (forall(y: G) { s.as_subgroup.contains(a.inverse * y) = s.as_subgroup.contains(b.inverse * y) }) implies
    finite_left_coset_set[G](s, a) = finite_left_coset_set[G](s, b)
} by {
    finite_left_coset_sets_pointwise_eq_from_raw[G](s, a, b)
    set_ext[G](finite_left_coset_set[G](s, a), finite_left_coset_set[G](s, b))
}

// Proven helper: finite_left_coset_sets_disjoint_from_raw_none

/// An auxiliary proposition for Lagrange's theorem.
theorem finite_left_coset_sets_disjoint_from_raw_none[G: FiniteGroup](s: FiniteSubgroup[G], a: G, b: G) {
    not(exists(x: G) { s.as_subgroup.contains(a.inverse * x) and s.as_subgroup.contains(b.inverse * x) }) implies
    finite_left_coset_set[G](s, a).is_disjoint(finite_left_coset_set[G](s, b))
} by {
    forall(y: G) {
        if finite_left_coset_set[G](s, a).contains(y) and finite_left_coset_set[G](s, b).contains(y) {
            finite_left_coset_set[G](s, a).contains(y) = s.contains(a.inverse * y)
            finite_left_coset_set[G](s, b).contains(y) = s.contains(b.inverse * y)
            s.contains(a.inverse * y)
            s.contains(b.inverse * y)
            s.contains(a.inverse * y) = s.as_subgroup.contains(a.inverse * y)
            s.contains(b.inverse * y) = s.as_subgroup.contains(b.inverse * y)
            s.as_subgroup.contains(a.inverse * y)
            s.as_subgroup.contains(b.inverse * y)
            exists(x: G) { x = y and s.as_subgroup.contains(a.inverse * x) and s.as_subgroup.contains(b.inverse * x) }
            exists(x: G) { s.as_subgroup.contains(a.inverse * x) and s.as_subgroup.contains(b.inverse * x) }
            false
        }
        not(finite_left_coset_set[G](s, a).contains(y) and finite_left_coset_set[G](s, b).contains(y))
    }
}

// Proven helper: finite_left_cosets_equal_or_disjoint

/// An auxiliary proposition for Lagrange's theorem.
theorem finite_left_cosets_equal_or_disjoint[G: FiniteGroup](s: FiniteSubgroup[G], a: G, b: G) {
    finite_left_coset_set[G](s, a) = finite_left_coset_set[G](s, b) or
    finite_left_coset_set[G](s, a).is_disjoint(finite_left_coset_set[G](s, b))
} by {
    left_cosets_equal_or_disjoint_raw_clause[G](s.as_subgroup, a, b)
    if forall(y: G) { s.as_subgroup.contains(a.inverse * y) = s.as_subgroup.contains(b.inverse * y) } {
        finite_left_coset_sets_eq_from_raw_pointwise[G](s, a, b)
    } else {
        finite_left_coset_sets_disjoint_from_raw_none[G](s, a, b)
    }
}

// Proven helper: finite_left_coset_set_contains_representative

/// An auxiliary proposition for Lagrange's theorem.
theorem finite_left_coset_set_contains_representative[G: FiniteGroup](s: FiniteSubgroup[G], a: G) {
    finite_left_coset_set[G](s, a).contains(a)
} by {
    left_coset_representative_mem_raw_clause[G](s.as_subgroup, a)
    s.as_subgroup.contains(a.inverse * a)
    finite_subgroup_contains_as_subgroup_eq[G](s, a.inverse * a)
    s.contains(a.inverse * a) = s.as_subgroup.contains(a.inverse * a)
    s.contains(a.inverse * a)
    finite_left_coset_set_contains_eq_local[G](s, a, a)
    finite_left_coset_set[G](s, a).contains(a) = s.contains(a.inverse * a)
    finite_left_coset_set[G](s, a).contains(a)
}

// Proven helper: finite_left_cosets_cover_universal

/// An auxiliary proposition for Lagrange's theorem.
theorem finite_left_cosets_cover_universal[G: FiniteGroup](s: FiniteSubgroup[G]) {
    list_indexed_union[G, G](G.elements, function(a: G) { finite_left_coset_set[G](s, a) }) = Set[G].universal_set
} by {
    forall(x: G) {
        G.elements.contains_every
        G.elements.contains(x)
        finite_left_coset_set_contains_representative[G](s, x)
        list_indexed_union_contains_of_contains[G, G](G.elements, function(a: G) { finite_left_coset_set[G](s, a) }, x, x)
        list_indexed_union[G, G](G.elements, function(a: G) { finite_left_coset_set[G](s, a) }).contains(x)
    }
    set_eq_universal_of_forall_contains[G](list_indexed_union[G, G](G.elements, function(a: G) { finite_left_coset_set[G](s, a) }))
}

/// The auxiliary finite left coset representatives object used in Lagrange's theorem.
define finite_left_coset_representatives[G: FiniteGroup](s: FiniteSubgroup[G]) -> List[Set[G]] {
    map[G, Set[G]](G.elements, function(a: G) { finite_left_coset_set[G](s, a) }).unique
}

// Proven helper: finite_left_coset_representatives_is_unique
/// An auxiliary proposition for Lagrange's theorem.
theorem finite_left_coset_representatives_is_unique[G: FiniteGroup](s: FiniteSubgroup[G]) {
    finite_left_coset_representatives[G](s).is_unique
} by {
    let raw: List[Set[G]] = map[G, Set[G]](G.elements, function(a: G) { finite_left_coset_set[G](s, a) })
    unique_list_is_unique[Set[G]](raw)
}

// Proven helper: finite_left_coset_representatives_contains_coset
/// An auxiliary proposition for Lagrange's theorem.
theorem finite_left_coset_representatives_contains_coset[G: FiniteGroup](s: FiniteSubgroup[G], a: G) {
    finite_left_coset_representatives[G](s).contains(finite_left_coset_set[G](s, a))
} by {
    let raw: List[Set[G]] = map[G, Set[G]](G.elements, function(x: G) { finite_left_coset_set[G](s, x) })
    G.elements.contains(a)
    raw.contains(finite_left_coset_set[G](s, a))
    raw.unique.contains(finite_left_coset_set[G](s, a)) = raw.contains(finite_left_coset_set[G](s, a))
    raw.unique.contains(finite_left_coset_set[G](s, a))
}

// Proven helper: finite_left_coset_representatives_cover_universal
/// An auxiliary proposition for Lagrange's theorem.
theorem finite_left_coset_representatives_cover_universal[G: FiniteGroup](s: FiniteSubgroup[G]) {
    list_indexed_union[G, Set[G]](
        finite_left_coset_representatives[G](s),
        function(c: Set[G]) { c }
    ) = Set[G].universal_set
} by {
    forall(x: G) {
        finite_left_coset_representatives_contains_coset[G](s, x)
        finite_left_coset_set_contains_representative[G](s, x)
        list_indexed_union_contains_of_contains[G, Set[G]](
            finite_left_coset_representatives[G](s),
            function(c: Set[G]) { c },
            finite_left_coset_set[G](s, x),
            x
        )
        list_indexed_union[G, Set[G]](
            finite_left_coset_representatives[G](s),
            function(c: Set[G]) { c }
        ).contains(x)
    }
    set_eq_universal_of_forall_contains[G](
        list_indexed_union[G, Set[G]](
            finite_left_coset_representatives[G](s),
            function(c: Set[G]) { c }
        )
    )
}

// Proven helper: finite_left_coset_representative_cardinality_is_order
/// An auxiliary proposition for Lagrange's theorem.
theorem finite_left_coset_representative_cardinality_is_order[G: FiniteGroup](s: FiniteSubgroup[G], c: Set[G]) {
    finite_left_coset_representatives[G](s).contains(c) implies c.cardinality_is(s.order)
} by {
    let raw: List[Set[G]] = map[G, Set[G]](G.elements, function(a: G) { finite_left_coset_set[G](s, a) })
    finite_left_coset_representatives[G](s) = raw.unique
    unique_preserves_contains[Set[G]](raw, c)
    raw.unique.contains(c) = raw.contains(c)
    raw.contains(c)
    map_contains[G, Set[G]](G.elements, function(a: G) { finite_left_coset_set[G](s, a) }, c)
    let a: G satisfy {
        G.elements.contains(a) and finite_left_coset_set[G](s, a) = c
    }
    finite_left_coset_set_cardinality_is_order[G](s, a)
    c = finite_left_coset_set[G](s, a)
    c.cardinality_is(s.order)
}

// Proven helper: finite_left_coset_representatives_distinct_disjoint
/// An auxiliary proposition for Lagrange's theorem.
theorem finite_left_coset_representatives_distinct_disjoint[G: FiniteGroup](s: FiniteSubgroup[G], c: Set[G], d: Set[G]) {
    finite_left_coset_representatives[G](s).contains(c) and
    finite_left_coset_representatives[G](s).contains(d) and
    c != d
    implies c.is_disjoint(d)
} by {
    let raw: List[Set[G]] = map[G, Set[G]](G.elements, function(a: G) { finite_left_coset_set[G](s, a) })
    raw.unique.contains(c)
    raw.unique.contains(d)
    unique_contains_imp_contains[Set[G]](raw, c)
    unique_contains_imp_contains[Set[G]](raw, d)
    raw.contains(c)
    raw.contains(d)
    map_contains[G, Set[G]](G.elements, function(a: G) { finite_left_coset_set[G](s, a) }, c)
    map_contains[G, Set[G]](G.elements, function(a: G) { finite_left_coset_set[G](s, a) }, d)
    let a: G satisfy { G.elements.contains(a) and finite_left_coset_set[G](s, a) = c }
    let b: G satisfy { G.elements.contains(b) and finite_left_coset_set[G](s, b) = d }
    finite_left_cosets_equal_or_disjoint[G](s, a, b)
    if finite_left_coset_set[G](s, a) = finite_left_coset_set[G](s, b) {
        c = d
        false
    }
    if finite_left_coset_set[G](s, a).is_disjoint(finite_left_coset_set[G](s, b)) {
        c.is_disjoint(d)
    }
}

from nat import divides_mul, divides_self

from data.basic.set import cardinality_is_well_defined, list_set, list_set_contains_eq, unique_list_set_cardinality_is_length, list_indexed_union_contains_eq

/// An auxiliary proposition for Lagrange's theorem.
theorem finite_group_universal_set_cardinality_is_order[G: FiniteGroup] {
    Set[G].universal_set.cardinality_is(G.order)
} by {
    forall(x: G) {
        G.elements.contains(x)
        list_set_contains_eq[G](G.elements, x)
        list_set[G](G.elements).contains(x)
    }
    set_eq_universal_of_forall_contains[G](list_set[G](G.elements))
    list_set[G](G.elements) = Set[G].universal_set
    G.elements.is_unique
    unique_list_set_cardinality_is_length[G](G.elements)
    list_set[G](G.elements).cardinality_is(G.elements.length)
    G.order = G.elements.length
    Set[G].universal_set.cardinality_is(G.order)
}

// Proven helper: set_list_union_contains_head

/// An auxiliary proposition for Lagrange's theorem.
theorem finite_left_coset_representatives_all_cardinality_is_order[G: FiniteGroup](s: FiniteSubgroup[G]) {
    set_list_all_cardinality_is[G](finite_left_coset_representatives[G](s), s.order)
} by {
    forall(c: Set[G]) {
        finite_left_coset_representatives[G](s).contains(c) implies c.cardinality_is(s.order)
        finite_left_coset_representative_cardinality_is_order[G](s, c)
    }
}

// Proven helper: finite_left_coset_representatives_pairwise_disjoint

/// An auxiliary proposition for Lagrange's theorem.
theorem finite_left_coset_representatives_pairwise_disjoint[G: FiniteGroup](s: FiniteSubgroup[G]) {
    set_list_pairwise_disjoint[G](finite_left_coset_representatives[G](s))
} by {
    forall(c: Set[G], d: Set[G]) {
        if finite_left_coset_representatives[G](s).contains(c) and finite_left_coset_representatives[G](s).contains(d) and c != d {
            finite_left_coset_representatives_distinct_disjoint[G](s, c, d)
            c.is_disjoint(d)
        }
        finite_left_coset_representatives[G](s).contains(c) and finite_left_coset_representatives[G](s).contains(d) and c != d implies c.is_disjoint(d)
    }
}

// Proven helper: finite_left_coset_representatives_recursively_disjoint

/// An auxiliary proposition for Lagrange's theorem.
theorem finite_left_coset_representatives_recursively_disjoint[G: FiniteGroup](s: FiniteSubgroup[G]) {
    set_list_recursively_disjoint[G](finite_left_coset_representatives[G](s))
} by {
    finite_left_coset_representatives_is_unique[G](s)
    finite_left_coset_representatives_pairwise_disjoint[G](s)
    unique_pairwise_implies_recursively_disjoint[G](finite_left_coset_representatives[G](s))
}

// Proven helper: finite_left_coset_representatives_union_cardinality

/// An auxiliary proposition for Lagrange's theorem.
theorem finite_left_coset_representatives_union_cardinality[G: FiniteGroup](s: FiniteSubgroup[G]) {
    set_list_union[G](finite_left_coset_representatives[G](s)).cardinality_is(s.order * finite_left_coset_representatives[G](s).length)
} by {
    finite_left_coset_representatives_all_cardinality_is_order[G](s)
    finite_left_coset_representatives_recursively_disjoint[G](s)
    set_list_union_cardinality_recursive[G](finite_left_coset_representatives[G](s), s.order)
}

// Proven helper: cons_contains_tail_if_ne

/// An auxiliary proposition for Lagrange's theorem.
theorem finite_left_coset_representatives_recursive_union_covers_universal[G: FiniteGroup](s: FiniteSubgroup[G]) {
    set_list_union[G](finite_left_coset_representatives[G](s)) = Set[G].universal_set
} by {
    forall(x: G) {
        finite_left_coset_representatives_cover_universal[G](s)
        list_indexed_union[G, Set[G]](
            finite_left_coset_representatives[G](s),
            function(c: Set[G]) { c }
        ).contains(x)
        list_indexed_union_contains_eq[G, Set[G]](
            finite_left_coset_representatives[G](s),
            function(c: Set[G]) { c },
            x
        )
        let c: Set[G] satisfy {
            finite_left_coset_representatives[G](s).contains(c) and c.contains(x)
        }
        set_list_union_contains_of_member[G](finite_left_coset_representatives[G](s), c, x)
        set_list_union[G](finite_left_coset_representatives[G](s)).contains(x)
    }
    set_eq_universal_of_forall_contains[G](set_list_union[G](finite_left_coset_representatives[G](s)))
}

/// An auxiliary proposition for Lagrange's theorem.
theorem finite_left_coset_representatives_order_product_eq_group_order[G: FiniteGroup](s: FiniteSubgroup[G]) {
    s.order * finite_left_coset_representatives[G](s).length = G.order
} by {
    let u: Set[G] = set_list_union[G](finite_left_coset_representatives[G](s))
    let n1: Nat = s.order * finite_left_coset_representatives[G](s).length
    let n2: Nat = G.order
    finite_left_coset_representatives_union_cardinality[G](s)
    finite_left_coset_representatives_recursive_union_covers_universal[G](s)
    finite_group_universal_set_cardinality_is_order[G]
    u.cardinality_is(n1)
    u.cardinality_is(n2)
    cardinality_is_well_defined[G](u, n1, n2)
}

/// An auxiliary proposition for Lagrange's theorem.
theorem top100_071_lagrange_theorem[G: FiniteGroup](s: FiniteSubgroup[G]) {
    s.order.divides(G.order)
} by {
    let reps_len: Nat = finite_left_coset_representatives[G](s).length
    divides_self(s.order)
    divides_mul(s.order, s.order, reps_len)
    finite_left_coset_representatives_order_product_eq_group_order[G](s)
}
