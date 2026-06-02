from comm_ring import CommRing, binomial, binomial_term
from list import partial
from nat import Nat

/// Freek Top 100 #44: the binomial theorem for commutative rings.
theorem top100_044_binomial_theorem[R: CommRing](a: R, b: R, n: Nat) {
    (a + b).pow(n) = partial(binomial_term[R](a, b, n), n.suc)
} by {
    binomial[R](a, b, n)
}
