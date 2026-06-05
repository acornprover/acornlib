from real import Real
from real import mul_nonneg
from order import lte_antisymm
from geometry import Point2, point2_on_circle_eq_dist_sq,
    point2_intersecting_chords_dist_sq_product_eq
from top100.theorem_091_triangle_inequality import real_le_of_square_le_square_nonneg

numerals Real

/// True when `length` is the Euclidean length of a real coordinate vector.
define point2_real_length_witness(p: Point2[Real], length: Real) -> Bool {
    Real.0 <= length and length * length = p.norm_sq
}

/// Nonnegative real numbers with equal squares are equal.
theorem real_eq_of_square_eq_square_nonneg_55(a: Real, b: Real) {
    Real.0 <= a and Real.0 <= b and a * a = b * b implies a = b
} by {
    if Real.0 <= a and Real.0 <= b and a * a = b * b {
        a * a <= b * b
        real_le_of_square_le_square_nonneg(a, b)
        a <= b
        b * b <= a * a
        real_le_of_square_le_square_nonneg(b, a)
        b <= a
        lte_antisymm(a, b)
        a = b
    }
}

/// Freek Top 100 #55: the products of the two segment lengths of intersecting chords are equal.
theorem top100_055_product_of_segments_of_chords(
    center: Point2[Real], radius_sq: Real,
    q: Point2[Real], r: Point2[Real], s: Point2[Real], t: Point2[Real], x: Point2[Real],
    u: Real, v: Real,
    qx: Real, rx: Real, sx: Real, tx: Real
) {
    center.on_circle(radius_sq, q) and
    center.on_circle(radius_sq, r) and
    center.on_circle(radius_sq, s) and
    center.on_circle(radius_sq, t) and
    Real.0 <= u and u <= Real.1 and
    Real.0 <= v and v <= Real.1 and
    x = q.param_line(r, u) and
    x = s.param_line(t, v) and
    point2_real_length_witness(q.sub(x), qx) and
    point2_real_length_witness(r.sub(x), rx) and
    point2_real_length_witness(s.sub(x), sx) and
    point2_real_length_witness(t.sub(x), tx)
    implies qx * rx = sx * tx
} by {
    if center.on_circle(radius_sq, q) and
        center.on_circle(radius_sq, r) and
        center.on_circle(radius_sq, s) and
        center.on_circle(radius_sq, t) and
        Real.0 <= u and u <= Real.1 and
        Real.0 <= v and v <= Real.1 and
        x = q.param_line(r, u) and
        x = s.param_line(t, v) and
        point2_real_length_witness(q.sub(x), qx) and
        point2_real_length_witness(r.sub(x), rx) and
        point2_real_length_witness(s.sub(x), sx) and
        point2_real_length_witness(t.sub(x), tx) {
        center.on_circle(radius_sq, q) and
            center.on_circle(radius_sq, r) and
            center.on_circle(radius_sq, s) and
            center.on_circle(radius_sq, t) and
            x = q.param_line(r, u) and
            x = s.param_line(t, v)
        point2_intersecting_chords_dist_sq_product_eq(center, radius_sq, q, r, s, t, x, u, v)
        q.dist_sq(x) * r.dist_sq(x) = s.dist_sq(x) * t.dist_sq(x)

        Real.0 <= qx
        Real.0 <= rx
        Real.0 <= sx
        Real.0 <= tx
        qx >= Real.0
        rx >= Real.0
        sx >= Real.0
        tx >= Real.0
        qx * qx = q.sub(x).norm_sq
        rx * rx = r.sub(x).norm_sq
        sx * sx = s.sub(x).norm_sq
        tx * tx = t.sub(x).norm_sq
        q.dist_sq(x) = q.sub(x).norm_sq
        r.dist_sq(x) = r.sub(x).norm_sq
        s.dist_sq(x) = s.sub(x).norm_sq
        t.dist_sq(x) = t.sub(x).norm_sq
        qx * qx = q.dist_sq(x)
        rx * rx = r.dist_sq(x)
        sx * sx = s.dist_sq(x)
        tx * tx = t.dist_sq(x)

        let left = qx * rx
        let right = sx * tx
        mul_nonneg(qx, rx)
        left >= Real.0
        Real.0 <= left
        mul_nonneg(sx, tx)
        right >= Real.0
        Real.0 <= right
        left * left = (qx * qx) * (rx * rx)
        right * right = (sx * sx) * (tx * tx)
        left * left = q.dist_sq(x) * r.dist_sq(x)
        right * right = s.dist_sq(x) * t.dist_sq(x)
        left * left = right * right
        Real.0 <= left and Real.0 <= right and left * left = right * right
        real_eq_of_square_eq_square_nonneg_55(left, right)
        left = right
        qx * rx = sx * tx
    }
}
