from combinatorics import binom
from nat import Nat
from binary_words import successful_ballot_words
from ballot_reflection import successful_ballot_words_suc_suc_length_binom_difference

numerals Nat

/// The Ballot Problem, in binomial-difference form.
theorem top100_030_the_ballot_problem(positives: Nat, negatives: Nat) {
    negatives <= positives implies
        successful_ballot_words(positives.suc, negatives.suc).length +
            (positives + negatives.suc).binom(positives.suc) =
            (positives + negatives.suc).binom(positives)
} by {
    successful_ballot_words_suc_suc_length_binom_difference(positives, negatives)
}
