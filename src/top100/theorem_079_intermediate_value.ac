from order_set import closed_interval_set, closed_interval_set_lower_le,
    closed_interval_set_le_upper
from real import Real, continuous, intermediate_value_closed_interval

/// Freek Top 100 #79: the intermediate value theorem on a closed interval.
theorem top100_079_intermediate_value(
    f: Real -> Real, lower: Real, upper: Real, target: Real
) {
    continuous(f) and lower <= upper and f(lower) <= target and target <= f(upper)
    implies exists(point: Real) {
        lower <= point and point <= upper and f(point) = target
    }
} by {
    if continuous(f) and lower <= upper and f(lower) <= target and target <= f(upper) {
        intermediate_value_closed_interval(f, lower, upper, target)
        exists(point: Real) {
            closed_interval_set(lower, upper).contains(point) and f(point) = target
        }
        let point: Real satisfy {
            closed_interval_set(lower, upper).contains(point) and f(point) = target
        }
        closed_interval_set(lower, upper).contains(point)
        f(point) = target
        closed_interval_set_lower_le(lower, upper, point)
        lower <= point
        closed_interval_set_le_upper(lower, upper, point)
        point <= upper
        lower <= point and point <= upper and f(point) = target
        exists(w: Real) {
            lower <= w and w <= upper and f(w) = target
        }
    }
}
