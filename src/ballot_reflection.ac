from combinatorics import binom
from nat import Nat
from list import List, filter_contained_by_and, filter_contains_and, map,
    unique_implies_tail_unique
from list import is_permutation, permutation_preserves_length,
    unique_same_contains_imp_permutation
from list import map_contains, map_contains_of_contains, map_length
from binary_words import Vote, bad_ballot_words, bad_ballot_words_unique,
    ballot_prefixes_negative, ballot_prefixes_nonnegative, ballot_prefixes_nonnegative_from,
    ballot_prefixes_nonnegative_from_cons_negative_elim,
    ballot_prefixes_nonnegative_from_cons_negative_intro,
    ballot_prefixes_nonnegative_from_cons_positive_elim,
    ballot_prefixes_nonnegative_from_cons_positive_intro, ballot_word_has_counts,
    ballot_words, ballot_words_complete, ballot_words_length_binom, ballot_words_sound,
    ballot_words_unique,
    cons_unique_of_tail_unique_not_contains, negative_count, negative_count_cons_negative,
    negative_count_cons_positive, positive_count, positive_count_cons_negative,
    positive_count_cons_positive, successful_ballot_words,
    successful_ballot_words_suc_length_eq_weak, unique_cons_not_contains,
    weak_bad_ballot_words_partition_length, weak_ballot_words

numerals Nat

/// Reflects a ballot word up to the first prefix where negatives exceed positives.
define reflect_first_bad_from(
    xs: List[Vote],
    positives_so_far: Nat,
    negatives_so_far: Nat
) -> List[Vote] {
    match xs {
        List.nil[Vote] {
            List.nil[Vote]
        }
        List.cons(head, tail) {
            match head {
                Vote.positive {
                    if negatives_so_far <= positives_so_far.suc {
                        List.cons(
                            Vote.negative,
                            reflect_first_bad_from(tail, positives_so_far.suc, negatives_so_far)
                        )
                    } else {
                        List.cons(Vote.negative, tail)
                    }
                }
                Vote.negative {
                    if negatives_so_far.suc <= positives_so_far {
                        List.cons(
                            Vote.positive,
                            reflect_first_bad_from(tail, positives_so_far, negatives_so_far.suc)
                        )
                    } else {
                        List.cons(Vote.positive, tail)
                    }
                }
            }
        }
    }
}

/// Reflects a ballot word up to the first bad prefix from zero balance.
define reflect_first_bad(xs: List[Vote]) -> List[Vote] {
    reflect_first_bad_from(xs, Nat.0, Nat.0)
}

/// True if the running positive-minus-negative gap never becomes negative.
define prefixes_nonnegative_gap(xs: List[Vote], gap: Nat) -> Bool {
    match xs {
        List.nil[Vote] {
            true
        }
        List.cons(head, tail) {
            match head {
                Vote.positive {
                    prefixes_nonnegative_gap(tail, gap.suc)
                }
                Vote.negative {
                    if gap = Nat.0 {
                        false
                    } else {
                        prefixes_nonnegative_gap(tail, gap - Nat.1)
                    }
                }
            }
        }
    }
}

/// Reflects a ballot word up to the first prefix where the gap would become negative.
define reflect_first_bad_gap(xs: List[Vote], gap: Nat) -> List[Vote] {
    match xs {
        List.nil[Vote] {
            List.nil[Vote]
        }
        List.cons(head, tail) {
            match head {
                Vote.positive {
                    List.cons(Vote.negative, reflect_first_bad_gap(tail, gap.suc))
                }
                Vote.negative {
                    if gap = Nat.0 {
                        List.cons(Vote.positive, tail)
                    } else {
                        List.cons(Vote.positive, reflect_first_bad_gap(tail, gap - Nat.1))
                    }
                }
            }
        }
    }
}

/// Gap-based reflection from zero balance.
define reflect_first_bad_gap_zero(xs: List[Vote]) -> List[Vote] {
    reflect_first_bad_gap(xs, Nat.0)
}

/// The tail induction hypothesis needed for reflection length preservation.
define reflect_first_bad_from_length_hyp(xs: List[Vote]) -> Bool {
    forall(a: Nat, b: Nat) {
        reflect_first_bad_from(xs, a, b).length = xs.length
    }
}

/// Reflection preserves length when a positive vote is still before the first bad prefix.
theorem reflect_first_bad_from_cons_positive_continue_length(
    tail: List[Vote],
    positives_so_far: Nat,
    negatives_so_far: Nat
) {
    (negatives_so_far <= positives_so_far.suc and
        reflect_first_bad_from(tail, positives_so_far.suc, negatives_so_far).length = tail.length
    ) implies reflect_first_bad_from(List.cons(Vote.positive, tail), positives_so_far, negatives_so_far).length =
        List.cons(Vote.positive, tail).length
} by {
    if negatives_so_far <= positives_so_far.suc and
        reflect_first_bad_from(tail, positives_so_far.suc, negatives_so_far).length = tail.length {
        reflect_first_bad_from(List.cons(Vote.positive, tail), positives_so_far, negatives_so_far) =
            List.cons(
                Vote.negative,
                reflect_first_bad_from(tail, positives_so_far.suc, negatives_so_far)
            )
        reflect_first_bad_from(List.cons(Vote.positive, tail), positives_so_far, negatives_so_far).length =
            tail.length.suc
        reflect_first_bad_from(List.cons(Vote.positive, tail), positives_so_far, negatives_so_far).length =
            List.cons(Vote.positive, tail).length
    }
}

/// Reflection preserves length when a positive vote is the first bad-prefix stop case.
theorem reflect_first_bad_from_cons_positive_stop_length(
    tail: List[Vote],
    positives_so_far: Nat,
    negatives_so_far: Nat
) {
    not (negatives_so_far <= positives_so_far.suc) implies
        reflect_first_bad_from(List.cons(Vote.positive, tail), positives_so_far, negatives_so_far).length =
            List.cons(Vote.positive, tail).length
} by {
    if not (negatives_so_far <= positives_so_far.suc) {
        reflect_first_bad_from(List.cons(Vote.positive, tail), positives_so_far, negatives_so_far) =
            List.cons(Vote.negative, tail)
        reflect_first_bad_from(List.cons(Vote.positive, tail), positives_so_far, negatives_so_far).length =
            List.cons(Vote.positive, tail).length
    }
}

/// Reflection preserves length when a negative vote is still before the first bad prefix.
theorem reflect_first_bad_from_cons_negative_continue_length(
    tail: List[Vote],
    positives_so_far: Nat,
    negatives_so_far: Nat
) {
    (negatives_so_far.suc <= positives_so_far and
        reflect_first_bad_from(tail, positives_so_far, negatives_so_far.suc).length = tail.length
    ) implies reflect_first_bad_from(List.cons(Vote.negative, tail), positives_so_far, negatives_so_far).length =
        List.cons(Vote.negative, tail).length
} by {
    if negatives_so_far.suc <= positives_so_far and
        reflect_first_bad_from(tail, positives_so_far, negatives_so_far.suc).length = tail.length {
        reflect_first_bad_from(List.cons(Vote.negative, tail), positives_so_far, negatives_so_far) =
            List.cons(
                Vote.positive,
                reflect_first_bad_from(tail, positives_so_far, negatives_so_far.suc)
            )
        reflect_first_bad_from(List.cons(Vote.negative, tail), positives_so_far, negatives_so_far).length =
            tail.length.suc
        reflect_first_bad_from(List.cons(Vote.negative, tail), positives_so_far, negatives_so_far).length =
            List.cons(Vote.negative, tail).length
    }
}

/// Reflection preserves length when a negative vote is the first bad-prefix stop case.
theorem reflect_first_bad_from_cons_negative_stop_length(
    tail: List[Vote],
    positives_so_far: Nat,
    negatives_so_far: Nat
) {
    not (negatives_so_far.suc <= positives_so_far) implies
        reflect_first_bad_from(List.cons(Vote.negative, tail), positives_so_far, negatives_so_far).length =
            List.cons(Vote.negative, tail).length
} by {
    if not (negatives_so_far.suc <= positives_so_far) {
        reflect_first_bad_from(List.cons(Vote.negative, tail), positives_so_far, negatives_so_far) =
            List.cons(Vote.positive, tail)
        reflect_first_bad_from(List.cons(Vote.negative, tail), positives_so_far, negatives_so_far).length =
            List.cons(Vote.negative, tail).length
    }
}

/// Reflection to the first bad prefix preserves length.
theorem reflect_first_bad_from_length(xs: List[Vote], positives_so_far: Nat, negatives_so_far: Nat) {
    reflect_first_bad_from(xs, positives_so_far, negatives_so_far).length = xs.length
} by {
    define p(ys: List[Vote]) -> Bool {
        forall(a: Nat, b: Nat) {
            reflect_first_bad_from(ys, a, b).length = ys.length
        }
    }

    p(List.nil[Vote])
    forall(head: Vote, tail: List[Vote]) {
        if p(tail) {
            forall(a: Nat, b: Nat) {
                match head {
                    Vote.positive {
                        head = Vote.positive
                        if b <= a.suc {
                            reflect_first_bad_from(tail, a.suc, b).length = tail.length
                            reflect_first_bad_from_cons_positive_continue_length(tail, a, b)
                            reflect_first_bad_from(List.cons(head, tail), a, b).length =
                                List.cons(head, tail).length
                        }
                        if not (b <= a.suc) {
                            reflect_first_bad_from_cons_positive_stop_length(tail, a, b)
                            reflect_first_bad_from(List.cons(head, tail), a, b).length =
                                List.cons(head, tail).length
                        }
                        reflect_first_bad_from(List.cons(head, tail), a, b).length =
                            List.cons(head, tail).length
                    }
                    Vote.negative {
                        head = Vote.negative
                        if b.suc <= a {
                            reflect_first_bad_from(tail, a, b.suc).length = tail.length
                            reflect_first_bad_from_cons_negative_continue_length(tail, a, b)
                            reflect_first_bad_from(List.cons(head, tail), a, b).length =
                                List.cons(head, tail).length
                        }
                        if not (b.suc <= a) {
                            reflect_first_bad_from_cons_negative_stop_length(tail, a, b)
                            reflect_first_bad_from(List.cons(head, tail), a, b).length =
                                List.cons(head, tail).length
                        }
                        reflect_first_bad_from(List.cons(head, tail), a, b).length =
                            List.cons(head, tail).length
                    }
                }
                reflect_first_bad_from(List.cons(head, tail), a, b).length =
                    List.cons(head, tail).length
            }
            p(List.cons(head, tail))
        }
    }

    p(xs)
}

/// Reflection to the first bad prefix preserves length from zero balance.
theorem reflect_first_bad_length(xs: List[Vote]) {
    reflect_first_bad(xs).length = xs.length
} by {
    reflect_first_bad_from_length(xs, Nat.0, Nat.0)
}

/// Gap-based reflection preserves length.
theorem reflect_first_bad_gap_length(xs: List[Vote], gap: Nat) {
    reflect_first_bad_gap(xs, gap).length = xs.length
} by {
    define p(ys: List[Vote]) -> Bool {
        forall(g: Nat) {
            reflect_first_bad_gap(ys, g).length = ys.length
        }
    }

    p(List.nil[Vote])
    forall(head: Vote, tail: List[Vote]) {
        if p(tail) {
            forall(g: Nat) {
                match head {
                    Vote.positive {
                        reflect_first_bad_gap(List.cons(head, tail), g) =
                            List.cons(Vote.negative, reflect_first_bad_gap(tail, g.suc))
                        reflect_first_bad_gap(tail, g.suc).length = tail.length
                        reflect_first_bad_gap(List.cons(head, tail), g).length = tail.length.suc
                        reflect_first_bad_gap(List.cons(head, tail), g).length = List.cons(head, tail).length
                    }
                    Vote.negative {
                        if g = Nat.0 {
                            reflect_first_bad_gap(List.cons(head, tail), g) = List.cons(Vote.positive, tail)
                            reflect_first_bad_gap(List.cons(head, tail), g).length = List.cons(head, tail).length
                        }
                        if g != Nat.0 {
                            reflect_first_bad_gap(List.cons(head, tail), g) =
                                List.cons(Vote.positive, reflect_first_bad_gap(tail, g - Nat.1))
                            reflect_first_bad_gap(tail, g - Nat.1).length = tail.length
                            reflect_first_bad_gap(List.cons(head, tail), g).length = tail.length.suc
                            reflect_first_bad_gap(List.cons(head, tail), g).length = List.cons(head, tail).length
                        }
                        reflect_first_bad_gap(List.cons(head, tail), g).length = List.cons(head, tail).length
                    }
                }
                reflect_first_bad_gap(List.cons(head, tail), g).length = List.cons(head, tail).length
            }
            p(List.cons(head, tail))
        }
    }

    p(xs)
}

/// Gap-based reflection from zero balance preserves length.
theorem reflect_first_bad_gap_zero_length(xs: List[Vote]) {
    reflect_first_bad_gap_zero(xs).length = xs.length
} by {
    reflect_first_bad_gap_length(xs, Nat.0)
}

/// Count shift for a positive vote before the first bad prefix.
theorem reflect_first_bad_gap_positive_continue_count_shift(tail: List[Vote], gap: Nat) {
    (positive_count(reflect_first_bad_gap(tail, gap.suc)) = positive_count(tail) + gap.suc.suc and
        negative_count(reflect_first_bad_gap(tail, gap.suc)) + gap.suc.suc = negative_count(tail)
    ) implies (
        positive_count(reflect_first_bad_gap(List.cons(Vote.positive, tail), gap)) =
            positive_count(List.cons(Vote.positive, tail)) + gap.suc and
        negative_count(reflect_first_bad_gap(List.cons(Vote.positive, tail), gap)) + gap.suc =
            negative_count(List.cons(Vote.positive, tail))
    )
} by {
    if positive_count(reflect_first_bad_gap(tail, gap.suc)) = positive_count(tail) + gap.suc.suc and
        negative_count(reflect_first_bad_gap(tail, gap.suc)) + gap.suc.suc = negative_count(tail) {
        positive_count_cons_negative(reflect_first_bad_gap(tail, gap.suc))
        positive_count_cons_positive(tail)
        positive_count(tail).suc + gap.suc = positive_count(tail) + gap.suc.suc
        negative_count_cons_negative(reflect_first_bad_gap(tail, gap.suc))
        Nat.1 + negative_count(reflect_first_bad_gap(tail, gap.suc)) + gap.suc =
            negative_count(reflect_first_bad_gap(tail, gap.suc)) + gap.suc.suc
        negative_count_cons_positive(tail)
        negative_count(reflect_first_bad_gap(List.cons(Vote.positive, tail), gap)) + gap.suc =
            negative_count(List.cons(Vote.positive, tail))
        positive_count(reflect_first_bad_gap(List.cons(Vote.positive, tail), gap)) =
            positive_count(List.cons(Vote.positive, tail)) + gap.suc and
        negative_count(reflect_first_bad_gap(List.cons(Vote.positive, tail), gap)) + gap.suc =
            negative_count(List.cons(Vote.positive, tail))
    }
}

/// Count shift for a negative vote that is immediately the first bad prefix.
theorem reflect_first_bad_gap_negative_zero_count_shift(tail: List[Vote]) {
    positive_count(reflect_first_bad_gap(List.cons(Vote.negative, tail), Nat.0)) =
        positive_count(List.cons(Vote.negative, tail)).suc and
    negative_count(reflect_first_bad_gap(List.cons(Vote.negative, tail), Nat.0)).suc =
        negative_count(List.cons(Vote.negative, tail))
} by {
    reflect_first_bad_gap(List.cons(Vote.negative, tail), Nat.0) = List.cons(Vote.positive, tail)
    positive_count_cons_positive(tail)
    positive_count_cons_negative(tail)
    negative_count_cons_positive(tail)
    negative_count_cons_negative(tail)
    negative_count(reflect_first_bad_gap(List.cons(Vote.negative, tail), Nat.0)).suc =
        negative_count(List.cons(Vote.negative, tail))
}

/// Count shift for a negative vote before the first bad prefix.
theorem reflect_first_bad_gap_negative_continue_count_shift(tail: List[Vote], gap: Nat) {
    (gap != Nat.0 and
        positive_count(reflect_first_bad_gap(tail, gap - Nat.1)) = positive_count(tail) + (gap - Nat.1).suc and
        negative_count(reflect_first_bad_gap(tail, gap - Nat.1)) + (gap - Nat.1).suc = negative_count(tail)
    ) implies (
        positive_count(reflect_first_bad_gap(List.cons(Vote.negative, tail), gap)) =
            positive_count(List.cons(Vote.negative, tail)) + gap.suc and
        negative_count(reflect_first_bad_gap(List.cons(Vote.negative, tail), gap)) + gap.suc =
            negative_count(List.cons(Vote.negative, tail))
    )
} by {
    if gap != Nat.0 and
        positive_count(reflect_first_bad_gap(tail, gap - Nat.1)) = positive_count(tail) + (gap - Nat.1).suc and
        negative_count(reflect_first_bad_gap(tail, gap - Nat.1)) + (gap - Nat.1).suc = negative_count(tail) {
        let (pred: Nat) satisfy { pred.suc = gap }
        (gap - Nat.1).suc = gap
        reflect_first_bad_gap(List.cons(Vote.negative, tail), gap) =
            List.cons(Vote.positive, reflect_first_bad_gap(tail, gap - Nat.1))
        positive_count_cons_positive(reflect_first_bad_gap(tail, gap - Nat.1))
        positive_count_cons_negative(tail)
        negative_count_cons_positive(reflect_first_bad_gap(tail, gap - Nat.1))
        negative_count_cons_negative(tail)
        negative_count(reflect_first_bad_gap(List.cons(Vote.negative, tail), gap)) + gap.suc =
            negative_count(List.cons(Vote.negative, tail))
        positive_count(reflect_first_bad_gap(List.cons(Vote.negative, tail), gap)) =
            positive_count(List.cons(Vote.negative, tail)) + gap.suc and
        negative_count(reflect_first_bad_gap(List.cons(Vote.negative, tail), gap)) + gap.suc =
            negative_count(List.cons(Vote.negative, tail))
    }
}

/// True if gap-based reflection has shifted one more vote than the starting gap.
define gap_reflection_count_shifted(xs: List[Vote], gap: Nat) -> Bool {
    positive_count(reflect_first_bad_gap(xs, gap)) = positive_count(xs) + gap.suc and
        negative_count(reflect_first_bad_gap(xs, gap)) + gap.suc = negative_count(xs)
}

/// The positive continuation step preserves the packaged count-shift predicate.
theorem gap_reflection_count_shifted_cons_positive(tail: List[Vote], gap: Nat) {
    gap_reflection_count_shifted(tail, gap.suc) implies
        gap_reflection_count_shifted(List.cons(Vote.positive, tail), gap)
} by {
    if gap_reflection_count_shifted(tail, gap.suc) {
        positive_count(reflect_first_bad_gap(tail, gap.suc)) = positive_count(tail) + gap.suc.suc
        negative_count(reflect_first_bad_gap(tail, gap.suc)) + gap.suc.suc = negative_count(tail)
        reflect_first_bad_gap_positive_continue_count_shift(tail, gap)
        gap_reflection_count_shifted(List.cons(Vote.positive, tail), gap)
    }
}

/// The first-bad negative step satisfies the packaged count-shift predicate.
theorem gap_reflection_count_shifted_cons_negative_zero(tail: List[Vote]) {
    gap_reflection_count_shifted(List.cons(Vote.negative, tail), Nat.0)
} by {
    reflect_first_bad_gap_negative_zero_count_shift(tail)
    positive_count(List.cons(Vote.negative, tail)).suc =
        positive_count(List.cons(Vote.negative, tail)) + Nat.0.suc
    negative_count(reflect_first_bad_gap(List.cons(Vote.negative, tail), Nat.0)) + Nat.0.suc =
        negative_count(List.cons(Vote.negative, tail))
}

/// The negative continuation step preserves the packaged count-shift predicate.
theorem gap_reflection_count_shifted_cons_negative_continue(tail: List[Vote], gap: Nat) {
    gap != Nat.0 and gap_reflection_count_shifted(tail, gap - Nat.1) implies
        gap_reflection_count_shifted(List.cons(Vote.negative, tail), gap)
} by {
    if gap != Nat.0 and gap_reflection_count_shifted(tail, gap - Nat.1) {
        positive_count(reflect_first_bad_gap(tail, gap - Nat.1)) =
            positive_count(tail) + (gap - Nat.1).suc
        negative_count(reflect_first_bad_gap(tail, gap - Nat.1)) + (gap - Nat.1).suc =
            negative_count(tail)
        reflect_first_bad_gap_negative_continue_count_shift(tail, gap)
        gap_reflection_count_shifted(List.cons(Vote.negative, tail), gap)
    }
}

/// Bad gap-based reflection satisfies the packaged count-shift predicate.
theorem gap_reflection_bad_count_shifted(xs: List[Vote], gap: Nat) {
    not prefixes_nonnegative_gap(xs, gap) implies gap_reflection_count_shifted(xs, gap)
} by {
    define p(ys: List[Vote]) -> Bool {
        forall(g: Nat) {
            not prefixes_nonnegative_gap(ys, g) implies gap_reflection_count_shifted(ys, g)
        }
    }

    p(List.nil[Vote])
    forall(head: Vote, tail: List[Vote]) {
        if p(tail) {
            forall(g: Nat) {
                if not prefixes_nonnegative_gap(List.cons(head, tail), g) {
                    match head {
                        Vote.positive {
                            not prefixes_nonnegative_gap(tail, g.suc)
                            gap_reflection_count_shifted(tail, g.suc)
                            gap_reflection_count_shifted_cons_positive(tail, g)
                            gap_reflection_count_shifted(List.cons(head, tail), g)
                        }
                        Vote.negative {
                            if g = Nat.0 {
                                gap_reflection_count_shifted_cons_negative_zero(tail)
                                gap_reflection_count_shifted(List.cons(head, tail), g)
                            }
                            if g != Nat.0 {
                                gap_reflection_count_shifted(tail, g - Nat.1)
                                gap_reflection_count_shifted_cons_negative_continue(tail, g)
                                gap_reflection_count_shifted(List.cons(head, tail), g)
                            }
                        }
                    }
                    if not gap_reflection_count_shifted(List.cons(head, tail), g) {
                        match head {
                            Vote.positive {
                                not prefixes_nonnegative_gap(tail, g.suc)
                                gap_reflection_count_shifted(tail, g.suc)
                                gap_reflection_count_shifted_cons_positive(tail, g)
                                false
                            }
                            Vote.negative {
                                if g = Nat.0 {
                                    gap_reflection_count_shifted_cons_negative_zero(tail)
                                    false
                                }
                                if g != Nat.0 {
                                    gap_reflection_count_shifted_cons_negative_continue(tail, g)
                                    false
                                }
                                false
                            }
                        }
                    }
                    gap_reflection_count_shifted(List.cons(head, tail), g)
                }
            }
            p(List.cons(head, tail))
        }
    }

    p(xs)
}

/// Gap-based reflection shifts one more negative vote than the starting gap into positive votes.
theorem reflect_first_bad_gap_bad_count_shift(xs: List[Vote], gap: Nat) {
    not prefixes_nonnegative_gap(xs, gap) implies (
        positive_count(reflect_first_bad_gap(xs, gap)) = positive_count(xs) + gap.suc and
        negative_count(reflect_first_bad_gap(xs, gap)) + gap.suc = negative_count(xs)
    )
} by {
    if not prefixes_nonnegative_gap(xs, gap) {
        gap_reflection_bad_count_shifted(xs, gap)
        negative_count(reflect_first_bad_gap(xs, gap)) + gap.suc = negative_count(xs)
    }
}

/// Zero-gap reflection of a bad word changes one negative vote into a positive vote.
theorem reflect_first_bad_gap_zero_bad_count_shift(xs: List[Vote]) {
    not prefixes_nonnegative_gap(xs, Nat.0) implies (
        positive_count(reflect_first_bad_gap_zero(xs)) = positive_count(xs).suc and
        negative_count(reflect_first_bad_gap_zero(xs)).suc = negative_count(xs)
    )
} by {
    if not prefixes_nonnegative_gap(xs, Nat.0) {
        reflect_first_bad_gap_bad_count_shift(xs, Nat.0)
        negative_count(reflect_first_bad_gap_zero(xs)).suc = negative_count(xs)
    }
}

/// A nonnegative gap implies the corresponding two-counter weak-prefix predicate.
theorem prefixes_nonnegative_gap_imp_from_offset(xs: List[Vote], offset: Nat, gap: Nat) {
    prefixes_nonnegative_gap(xs, gap) implies
        ballot_prefixes_nonnegative_from(xs, offset + gap, offset)
} by {
    define p(ys: List[Vote]) -> Bool {
        forall(o: Nat, g: Nat) {
            prefixes_nonnegative_gap(ys, g) implies
                ballot_prefixes_nonnegative_from(ys, o + g, o)
        }
    }

    p(List.nil[Vote])
    forall(head: Vote, tail: List[Vote]) {
        if p(tail) {
            forall(o: Nat, g: Nat) {
                if prefixes_nonnegative_gap(List.cons(head, tail), g) {
                    match head {
                        Vote.positive {
                            prefixes_nonnegative_gap(tail, g.suc)
                            ballot_prefixes_nonnegative_from(tail, o + g.suc, o)
                            o <= (o + g).suc
                            ballot_prefixes_nonnegative_from_cons_positive_intro(tail, o + g, o)
                            ballot_prefixes_nonnegative_from(List.cons(head, tail), o + g, o)
                        }
                        Vote.negative {
                            head = Vote.negative
                            if g = Nat.0 {
                                prefixes_nonnegative_gap(List.cons(head, tail), g) = false
                                false
                            }
                            if g != Nat.0 {
                                let (pred: Nat) satisfy { pred.suc = g }
                                g - Nat.1 = pred
                                (g - Nat.1).suc = g
                                prefixes_nonnegative_gap(tail, g - Nat.1)
                                ballot_prefixes_nonnegative_from(tail, o.suc + (g - Nat.1), o.suc)
                                o.suc + (g - Nat.1) = o + g
                                o.suc <= o + g
                                ballot_prefixes_nonnegative_from_cons_negative_intro(tail, o + g, o)
                                ballot_prefixes_nonnegative_from(List.cons(head, tail), o + g, o)
                            }
                            ballot_prefixes_nonnegative_from(List.cons(head, tail), o + g, o)
                        }
                    }
                    ballot_prefixes_nonnegative_from(List.cons(head, tail), o + g, o)
                }
            }
            p(List.cons(head, tail))
        }
    }

    p(xs)
}

/// A nonnegative zero gap implies the original weak-prefix predicate.
theorem prefixes_nonnegative_gap_zero_imp_ballot_prefixes_nonnegative(xs: List[Vote]) {
    prefixes_nonnegative_gap(xs, Nat.0) implies ballot_prefixes_nonnegative(xs)
} by {
    if prefixes_nonnegative_gap(xs, Nat.0) {
        prefixes_nonnegative_gap_imp_from_offset(xs, Nat.0, Nat.0)
        ballot_prefixes_nonnegative(xs)
    }
}

/// The two-counter weak-prefix predicate implies the corresponding nonnegative gap.
theorem ballot_prefixes_nonnegative_from_imp_gap(xs: List[Vote], offset: Nat, gap: Nat) {
    ballot_prefixes_nonnegative_from(xs, offset + gap, offset) implies
        prefixes_nonnegative_gap(xs, gap)
} by {
    define p(ys: List[Vote]) -> Bool {
        forall(o: Nat, g: Nat) {
            ballot_prefixes_nonnegative_from(ys, o + g, o) implies
                prefixes_nonnegative_gap(ys, g)
        }
    }

    p(List.nil[Vote])
    forall(head: Vote, tail: List[Vote]) {
        if p(tail) {
            forall(o: Nat, g: Nat) {
                if ballot_prefixes_nonnegative_from(List.cons(head, tail), o + g, o) {
                    match head {
                        Vote.positive {
                            ballot_prefixes_nonnegative_from_cons_positive_elim(tail, o + g, o)
                            ballot_prefixes_nonnegative_from(tail, o + g.suc, o)
                            prefixes_nonnegative_gap(List.cons(head, tail), g)
                        }
                        Vote.negative {
                            head = Vote.negative
                            if g = Nat.0 {
                                ballot_prefixes_nonnegative_from_cons_negative_elim(tail, o + g, o)
                                o.suc <= o + g
                                not o.suc <= o + g
                                false
                            }
                            if g != Nat.0 {
                                let (pred: Nat) satisfy { pred.suc = g }
                                g - Nat.1 = pred
                                (g - Nat.1).suc = g
                                ballot_prefixes_nonnegative_from_cons_negative_elim(tail, o + g, o)
                                ballot_prefixes_nonnegative_from(tail, o + g, o.suc)
                                o.suc + (g - Nat.1) = o + g
                                ballot_prefixes_nonnegative_from(tail, o.suc + (g - Nat.1), o.suc)
                                prefixes_nonnegative_gap(List.cons(head, tail), g)
                            }
                            prefixes_nonnegative_gap(List.cons(head, tail), g)
                        }
                    }
                    prefixes_nonnegative_gap(List.cons(head, tail), g)
                }
            }
            p(List.cons(head, tail))
        }
    }

    p(xs)
}

/// The original weak-prefix predicate implies a nonnegative zero gap.
theorem ballot_prefixes_nonnegative_imp_gap_zero(xs: List[Vote]) {
    ballot_prefixes_nonnegative(xs) implies prefixes_nonnegative_gap(xs, Nat.0)
} by {
    if ballot_prefixes_nonnegative(xs) {
        ballot_prefixes_nonnegative_from_imp_gap(xs, Nat.0, Nat.0)
        prefixes_nonnegative_gap(xs, Nat.0)
    }
}

/// A negative zero-gap prefix gives a negative prefix in the original predicate.
theorem not_gap_zero_imp_ballot_prefixes_negative(xs: List[Vote]) {
    not prefixes_nonnegative_gap(xs, Nat.0) implies ballot_prefixes_negative(xs)
} by {
    if not prefixes_nonnegative_gap(xs, Nat.0) {
        if ballot_prefixes_nonnegative(xs) {
            ballot_prefixes_nonnegative_imp_gap_zero(xs)
            false
        }
        ballot_prefixes_negative(xs)
    }
}

/// A word with a negative prefix has a negative zero-gap prefix.
theorem ballot_prefixes_negative_imp_not_gap_zero(xs: List[Vote]) {
    ballot_prefixes_negative(xs) implies not prefixes_nonnegative_gap(xs, Nat.0)
} by {
    if ballot_prefixes_negative(xs) {
        if prefixes_nonnegative_gap(xs, Nat.0) {
            prefixes_nonnegative_gap_zero_imp_ballot_prefixes_nonnegative(xs)
            false
        }
    }
}

/// The reflection image of bad ballot words with one extra negative vote.
define reflected_bad_ballot_words(positives: Nat, negatives: Nat) -> List[List[Vote]] {
    map(bad_ballot_words(positives, negatives.suc), reflect_first_bad_gap_zero)
}

/// A reflected bad word has one more positive vote and one fewer negative vote.
theorem reflected_bad_ballot_word_has_counts(positives: Nat, negatives: Nat, word: List[Vote]) {
    bad_ballot_words(positives, negatives.suc).contains(word) implies
        ballot_word_has_counts(reflect_first_bad_gap_zero(word), positives.suc, negatives)
} by {
    if bad_ballot_words(positives, negatives.suc).contains(word) {
        filter_contained_by_and[List[Vote]](
            ballot_words(positives, negatives.suc),
            ballot_prefixes_negative,
            word
        )
        ballot_words(positives, negatives.suc).contains(word)
        ballot_prefixes_negative(word)
        ballot_words_sound(positives, negatives.suc, word)
        positive_count(word) = positives
        ballot_prefixes_negative_imp_not_gap_zero(word)
        reflect_first_bad_gap_zero_bad_count_shift(word)
        negative_count(reflect_first_bad_gap_zero(word)).suc = negatives.suc
        negative_count(reflect_first_bad_gap_zero(word)) = negatives
        ballot_word_has_counts(reflect_first_bad_gap_zero(word), positives.suc, negatives)
    }
}

/// A reflected bad word is generated with the shifted vote counts.
theorem reflected_bad_ballot_word_contains_ballot_words(
    positives: Nat,
    negatives: Nat,
    word: List[Vote]
) {
    bad_ballot_words(positives, negatives.suc).contains(word) implies
        ballot_words(positives.suc, negatives).contains(reflect_first_bad_gap_zero(word))
} by {
    if bad_ballot_words(positives, negatives.suc).contains(word) {
        reflected_bad_ballot_word_has_counts(positives, negatives, word)
        ballot_words_complete(positives.suc, negatives, reflect_first_bad_gap_zero(word))
        ballot_words(positives.suc, negatives).contains(reflect_first_bad_gap_zero(word))
    }
}

/// The reflected bad-word image is contained in the shifted generated ballot words.
theorem reflected_bad_ballot_words_contained_by_ballot_words(
    positives: Nat,
    negatives: Nat,
    word: List[Vote]
) {
    reflected_bad_ballot_words(positives, negatives).contains(word) implies
        ballot_words(positives.suc, negatives).contains(word)
} by {
    if reflected_bad_ballot_words(positives, negatives).contains(word) {
        map_contains[List[Vote], List[Vote]](
            bad_ballot_words(positives, negatives.suc),
            reflect_first_bad_gap_zero,
            word
        )
        let source: List[Vote] satisfy {
            bad_ballot_words(positives, negatives.suc).contains(source) and
                reflect_first_bad_gap_zero(source) = word
        }
        reflected_bad_ballot_word_contains_ballot_words(positives, negatives, source)
        ballot_words(positives.suc, negatives).contains(word)
    }
}

/// True if the running negative-minus-positive deficit never becomes negative.
define prefixes_nonpositive_deficit(xs: List[Vote], deficit: Nat) -> Bool {
    match xs {
        List.nil[Vote] {
            true
        }
        List.cons(head, tail) {
            match head {
                Vote.positive {
                    if deficit = Nat.0 {
                        false
                    } else {
                        prefixes_nonpositive_deficit(tail, deficit - Nat.1)
                    }
                }
                Vote.negative {
                    prefixes_nonpositive_deficit(tail, deficit.suc)
                }
            }
        }
    }
}

/// Reflects a word up to the first prefix where positive votes exceed negative votes.
define reflect_first_positive_deficit(xs: List[Vote], deficit: Nat) -> List[Vote] {
    match xs {
        List.nil[Vote] {
            List.nil[Vote]
        }
        List.cons(head, tail) {
            match head {
                Vote.positive {
                    if deficit = Nat.0 {
                        List.cons(Vote.negative, tail)
                    } else {
                        List.cons(Vote.negative, reflect_first_positive_deficit(tail, deficit - Nat.1))
                    }
                }
                Vote.negative {
                    List.cons(Vote.positive, reflect_first_positive_deficit(tail, deficit.suc))
                }
            }
        }
    }
}

/// First-positive reflection from zero deficit.
define reflect_first_positive_deficit_zero(xs: List[Vote]) -> List[Vote] {
    reflect_first_positive_deficit(xs, Nat.0)
}

/// Count shift for a positive vote that is immediately the first positive-leading prefix.
theorem reflect_first_positive_deficit_positive_zero_count_shift(tail: List[Vote]) {
    positive_count(reflect_first_positive_deficit(List.cons(Vote.positive, tail), Nat.0)).suc =
        positive_count(List.cons(Vote.positive, tail)) and
    negative_count(reflect_first_positive_deficit(List.cons(Vote.positive, tail), Nat.0)) =
        negative_count(List.cons(Vote.positive, tail)).suc
} by {
    reflect_first_positive_deficit(List.cons(Vote.positive, tail), Nat.0) = List.cons(Vote.negative, tail)
    positive_count_cons_negative(tail)
    positive_count_cons_positive(tail)
    negative_count_cons_negative(tail)
    negative_count_cons_positive(tail)
    negative_count(reflect_first_positive_deficit(List.cons(Vote.positive, tail), Nat.0)) =
        negative_count(List.cons(Vote.positive, tail)).suc
}

/// Count shift for a positive vote before the first positive-leading prefix.
theorem reflect_first_positive_deficit_positive_continue_count_shift(tail: List[Vote], deficit: Nat) {
    (deficit != Nat.0 and
        positive_count(reflect_first_positive_deficit(tail, deficit - Nat.1)) + (deficit - Nat.1).suc =
            positive_count(tail) and
        negative_count(reflect_first_positive_deficit(tail, deficit - Nat.1)) =
            negative_count(tail) + (deficit - Nat.1).suc
    ) implies (
        positive_count(reflect_first_positive_deficit(List.cons(Vote.positive, tail), deficit)) + deficit.suc =
            positive_count(List.cons(Vote.positive, tail)) and
        negative_count(reflect_first_positive_deficit(List.cons(Vote.positive, tail), deficit)) =
            negative_count(List.cons(Vote.positive, tail)) + deficit.suc
    )
} by {
    if deficit != Nat.0 and
        positive_count(reflect_first_positive_deficit(tail, deficit - Nat.1)) + (deficit - Nat.1).suc =
            positive_count(tail) and
        negative_count(reflect_first_positive_deficit(tail, deficit - Nat.1)) =
            negative_count(tail) + (deficit - Nat.1).suc {
        let (pred: Nat) satisfy { pred.suc = deficit }
        (deficit - Nat.1).suc = deficit
        reflect_first_positive_deficit(List.cons(Vote.positive, tail), deficit) =
            List.cons(Vote.negative, reflect_first_positive_deficit(tail, deficit - Nat.1))
        positive_count_cons_negative(reflect_first_positive_deficit(tail, deficit - Nat.1))
        positive_count_cons_positive(tail)
        negative_count_cons_negative(reflect_first_positive_deficit(tail, deficit - Nat.1))
        negative_count_cons_positive(tail)
        negative_count(reflect_first_positive_deficit(List.cons(Vote.positive, tail), deficit)) =
            negative_count(List.cons(Vote.positive, tail)) + deficit.suc
        positive_count(reflect_first_positive_deficit(List.cons(Vote.positive, tail), deficit)) + deficit.suc =
            positive_count(List.cons(Vote.positive, tail)) and
        negative_count(reflect_first_positive_deficit(List.cons(Vote.positive, tail), deficit)) =
            negative_count(List.cons(Vote.positive, tail)) + deficit.suc
    }
}

/// Count shift for a negative vote before the first positive-leading prefix.
theorem reflect_first_positive_deficit_negative_continue_count_shift(tail: List[Vote], deficit: Nat) {
    (positive_count(reflect_first_positive_deficit(tail, deficit.suc)) + deficit.suc.suc =
        positive_count(tail) and
        negative_count(reflect_first_positive_deficit(tail, deficit.suc)) =
            negative_count(tail) + deficit.suc.suc
    ) implies (
        positive_count(reflect_first_positive_deficit(List.cons(Vote.negative, tail), deficit)) + deficit.suc =
            positive_count(List.cons(Vote.negative, tail)) and
        negative_count(reflect_first_positive_deficit(List.cons(Vote.negative, tail), deficit)) =
            negative_count(List.cons(Vote.negative, tail)) + deficit.suc
    )
} by {
    if positive_count(reflect_first_positive_deficit(tail, deficit.suc)) + deficit.suc.suc =
        positive_count(tail) and
        negative_count(reflect_first_positive_deficit(tail, deficit.suc)) =
            negative_count(tail) + deficit.suc.suc {
        positive_count_cons_positive(reflect_first_positive_deficit(tail, deficit.suc))
        Nat.1 + positive_count(reflect_first_positive_deficit(tail, deficit.suc)) + deficit.suc =
            positive_count(reflect_first_positive_deficit(tail, deficit.suc)) + deficit.suc.suc
        positive_count_cons_negative(tail)
        negative_count_cons_positive(reflect_first_positive_deficit(tail, deficit.suc))
        negative_count_cons_negative(tail)
        Nat.1 + negative_count(tail) + deficit.suc = negative_count(tail) + deficit.suc.suc
        negative_count(reflect_first_positive_deficit(List.cons(Vote.negative, tail), deficit)) =
            negative_count(List.cons(Vote.negative, tail)) + deficit.suc
        positive_count(reflect_first_positive_deficit(List.cons(Vote.negative, tail), deficit)) + deficit.suc =
            positive_count(List.cons(Vote.negative, tail)) and
        negative_count(reflect_first_positive_deficit(List.cons(Vote.negative, tail), deficit)) =
            negative_count(List.cons(Vote.negative, tail)) + deficit.suc
    }
}

/// True if first-positive reflection shifts one more vote than the starting deficit.
define positive_deficit_reflection_count_shifted(xs: List[Vote], deficit: Nat) -> Bool {
    positive_count(reflect_first_positive_deficit(xs, deficit)) + deficit.suc = positive_count(xs) and
        negative_count(reflect_first_positive_deficit(xs, deficit)) = negative_count(xs) + deficit.suc
}

/// The first-positive stop step satisfies the packaged count-shift predicate.
theorem positive_deficit_reflection_count_shifted_cons_positive_zero(tail: List[Vote]) {
    positive_deficit_reflection_count_shifted(List.cons(Vote.positive, tail), Nat.0)
} by {
    reflect_first_positive_deficit_positive_zero_count_shift(tail)
    positive_count(reflect_first_positive_deficit(List.cons(Vote.positive, tail), Nat.0)) + Nat.0.suc =
        positive_count(List.cons(Vote.positive, tail))
    negative_count(List.cons(Vote.positive, tail)).suc =
        negative_count(List.cons(Vote.positive, tail)) + Nat.0.suc
}

/// The positive continuation step preserves the packaged first-positive count shift.
theorem positive_deficit_reflection_count_shifted_cons_positive_continue(tail: List[Vote], deficit: Nat) {
    deficit != Nat.0 and positive_deficit_reflection_count_shifted(tail, deficit - Nat.1) implies
        positive_deficit_reflection_count_shifted(List.cons(Vote.positive, tail), deficit)
} by {
    if deficit != Nat.0 and positive_deficit_reflection_count_shifted(tail, deficit - Nat.1) {
        positive_count(reflect_first_positive_deficit(tail, deficit - Nat.1)) + (deficit - Nat.1).suc =
            positive_count(tail)
        negative_count(reflect_first_positive_deficit(tail, deficit - Nat.1)) =
            negative_count(tail) + (deficit - Nat.1).suc
        reflect_first_positive_deficit_positive_continue_count_shift(tail, deficit)
        positive_deficit_reflection_count_shifted(List.cons(Vote.positive, tail), deficit)
    }
}

/// The negative continuation step preserves the packaged first-positive count shift.
theorem positive_deficit_reflection_count_shifted_cons_negative(tail: List[Vote], deficit: Nat) {
    positive_deficit_reflection_count_shifted(tail, deficit.suc) implies
        positive_deficit_reflection_count_shifted(List.cons(Vote.negative, tail), deficit)
} by {
    if positive_deficit_reflection_count_shifted(tail, deficit.suc) {
        positive_count(reflect_first_positive_deficit(tail, deficit.suc)) + deficit.suc.suc =
            positive_count(tail)
        negative_count(reflect_first_positive_deficit(tail, deficit.suc)) =
            negative_count(tail) + deficit.suc.suc
        reflect_first_positive_deficit_negative_continue_count_shift(tail, deficit)
        positive_deficit_reflection_count_shifted(List.cons(Vote.negative, tail), deficit)
    }
}

/// First-positive reflection shifts one more positive vote than the starting deficit into negative votes.
theorem reflect_first_positive_deficit_count_shift(xs: List[Vote], deficit: Nat) {
    not prefixes_nonpositive_deficit(xs, deficit) implies
        positive_deficit_reflection_count_shifted(xs, deficit)
} by {
    define p(ys: List[Vote]) -> Bool {
        forall(d: Nat) {
            not prefixes_nonpositive_deficit(ys, d) implies
                positive_deficit_reflection_count_shifted(ys, d)
        }
    }

    p(List.nil[Vote])
    forall(head: Vote, tail: List[Vote]) {
        if p(tail) {
            forall(d: Nat) {
                if not prefixes_nonpositive_deficit(List.cons(head, tail), d) {
                    match head {
                        Vote.positive {
                            if d = Nat.0 {
                                positive_deficit_reflection_count_shifted_cons_positive_zero(tail)
                                positive_deficit_reflection_count_shifted(List.cons(head, tail), d)
                            }
                            if d != Nat.0 {
                                positive_deficit_reflection_count_shifted(tail, d - Nat.1)
                                positive_deficit_reflection_count_shifted_cons_positive_continue(tail, d)
                                positive_deficit_reflection_count_shifted(List.cons(head, tail), d)
                            }
                            positive_deficit_reflection_count_shifted(List.cons(head, tail), d)
                        }
                        Vote.negative {
                            not prefixes_nonpositive_deficit(tail, d.suc)
                            positive_deficit_reflection_count_shifted(tail, d.suc)
                            positive_deficit_reflection_count_shifted_cons_negative(tail, d)
                            positive_deficit_reflection_count_shifted(List.cons(head, tail), d)
                        }
                    }
                    positive_deficit_reflection_count_shifted(List.cons(head, tail), d)
                }
            }
            p(List.cons(head, tail))
        }
    }

    p(xs)
}

/// Zero-deficit first-positive reflection changes one positive vote into a negative vote.
theorem reflect_first_positive_deficit_zero_count_shift(xs: List[Vote]) {
    not prefixes_nonpositive_deficit(xs, Nat.0) implies (
        positive_count(reflect_first_positive_deficit_zero(xs)).suc = positive_count(xs) and
        negative_count(reflect_first_positive_deficit_zero(xs)) = negative_count(xs).suc
    )
} by {
    if not prefixes_nonpositive_deficit(xs, Nat.0) {
        reflect_first_positive_deficit_count_shift(xs, Nat.0)
        positive_count(reflect_first_positive_deficit_zero(xs)) + Nat.0.suc = positive_count(xs)
        negative_count(xs) + Nat.0.suc = negative_count(xs).suc
        negative_count(reflect_first_positive_deficit_zero(xs)) = negative_count(xs).suc
    }
}

/// If positive votes never lead, their count is bounded by the negative count and initial deficit.
theorem prefixes_nonpositive_deficit_count_bound(xs: List[Vote], deficit: Nat) {
    prefixes_nonpositive_deficit(xs, deficit) implies
        positive_count(xs) <= negative_count(xs) + deficit
} by {
    define p(ys: List[Vote]) -> Bool {
        forall(d: Nat) {
            prefixes_nonpositive_deficit(ys, d) implies
                positive_count(ys) <= negative_count(ys) + d
        }
    }

    forall(d: Nat) {
        if prefixes_nonpositive_deficit(List.nil[Vote], d) {
            Nat.0 <= Nat.0 + d
            positive_count(List.nil[Vote]) <= negative_count(List.nil[Vote]) + d
        }
    }
    p(List.nil[Vote])
    forall(head: Vote, tail: List[Vote]) {
        if p(tail) {
            forall(d: Nat) {
                if prefixes_nonpositive_deficit(List.cons(head, tail), d) {
                    match head {
                        Vote.positive {
                            if d = Nat.0 {
                                prefixes_nonpositive_deficit(List.cons(head, tail), d) = false
                                false
                            }
                            if d != Nat.0 {
                                let (pred: Nat) satisfy { pred.suc = d }
                                positive_count(tail) <= negative_count(tail) + (d - Nat.1)
                                positive_count_cons_positive(tail)
                                negative_count_cons_positive(tail)
                                positive_count(tail).suc <= (negative_count(tail) + (d - Nat.1)).suc
                                negative_count(List.cons(head, tail)) + d =
                                    (negative_count(tail) + (d - Nat.1)).suc
                                positive_count(List.cons(head, tail)) <= negative_count(List.cons(head, tail)) + d
                            }
                            positive_count(List.cons(head, tail)) <= negative_count(List.cons(head, tail)) + d
                        }
                        Vote.negative {
                            prefixes_nonpositive_deficit(tail, d.suc)
                            positive_count(tail) <= negative_count(tail) + d.suc
                            positive_count_cons_negative(tail)
                            negative_count_cons_negative(tail)
                            Nat.1 + negative_count(tail) + d = negative_count(tail) + d.suc
                            positive_count(List.cons(head, tail)) <= negative_count(List.cons(head, tail)) + d
                        }
                    }
                    positive_count(List.cons(head, tail)) <= negative_count(List.cons(head, tail)) + d
                }
            }
            p(List.cons(head, tail))
        }
    }

    p(xs)
}

/// A word with one more positive vote than a bounded negative count must have a positive-leading prefix.
theorem ballot_word_counts_imp_not_prefixes_nonpositive_deficit_zero(
    word: List[Vote],
    positives: Nat,
    negatives: Nat
) {
    ballot_word_has_counts(word, positives.suc, negatives) and negatives <= positives implies
        not prefixes_nonpositive_deficit(word, Nat.0)
} by {
    if ballot_word_has_counts(word, positives.suc, negatives) and negatives <= positives {
        positive_count(word) = positives.suc
        negative_count(word) = negatives
        if prefixes_nonpositive_deficit(word, Nat.0) {
            prefixes_nonpositive_deficit_count_bound(word, Nat.0)
            positive_count(word) <= negative_count(word) + Nat.0
            positive_count(word) <= negative_count(word)
            positives.suc <= negatives
            positives.suc <= positives
            false
        }
    }
}

/// First-positive reflection creates a bad prefix for the corresponding gap.
theorem reflect_first_positive_deficit_not_gap(xs: List[Vote], deficit: Nat) {
    not prefixes_nonpositive_deficit(xs, deficit) implies
        not prefixes_nonnegative_gap(reflect_first_positive_deficit(xs, deficit), deficit)
} by {
    define p(ys: List[Vote]) -> Bool {
        forall(d: Nat) {
            not prefixes_nonpositive_deficit(ys, d) implies
                not prefixes_nonnegative_gap(reflect_first_positive_deficit(ys, d), d)
        }
    }

    p(List.nil[Vote])
    forall(head: Vote, tail: List[Vote]) {
        if p(tail) {
            forall(d: Nat) {
                if not prefixes_nonpositive_deficit(List.cons(head, tail), d) {
                    match head {
                        Vote.positive {
                            if d = Nat.0 {
                                reflect_first_positive_deficit(List.cons(head, tail), d) =
                                    List.cons(Vote.negative, tail)
                                not prefixes_nonnegative_gap(reflect_first_positive_deficit(List.cons(head, tail), d), d)
                            }
                            if d != Nat.0 {
                                not prefixes_nonnegative_gap(
                                    reflect_first_positive_deficit(tail, d - Nat.1),
                                    d - Nat.1
                                )
                                reflect_first_positive_deficit(List.cons(head, tail), d) =
                                    List.cons(Vote.negative, reflect_first_positive_deficit(tail, d - Nat.1))
                                not prefixes_nonnegative_gap(reflect_first_positive_deficit(List.cons(head, tail), d), d)
                            }
                            not prefixes_nonnegative_gap(reflect_first_positive_deficit(List.cons(head, tail), d), d)
                        }
                        Vote.negative {
                            not prefixes_nonpositive_deficit(tail, d.suc)
                            reflect_first_positive_deficit(List.cons(head, tail), d) =
                                List.cons(Vote.positive, reflect_first_positive_deficit(tail, d.suc))
                            prefixes_nonnegative_gap(reflect_first_positive_deficit(List.cons(head, tail), d), d) =
                                prefixes_nonnegative_gap(reflect_first_positive_deficit(tail, d.suc), d.suc)
                            not prefixes_nonnegative_gap(reflect_first_positive_deficit(List.cons(head, tail), d), d)
                        }
                    }
                    not prefixes_nonnegative_gap(reflect_first_positive_deficit(List.cons(head, tail), d), d)
                }
            }
            p(List.cons(head, tail))
        }
    }

    p(xs)
}

/// Zero-deficit first-positive reflection creates a negative prefix in the original predicate.
theorem reflect_first_positive_deficit_zero_prefixes_negative(xs: List[Vote]) {
    not prefixes_nonpositive_deficit(xs, Nat.0) implies
        ballot_prefixes_negative(reflect_first_positive_deficit_zero(xs))
} by {
    if not prefixes_nonpositive_deficit(xs, Nat.0) {
        reflect_first_positive_deficit_not_gap(xs, Nat.0)
        not_gap_zero_imp_ballot_prefixes_negative(reflect_first_positive_deficit_zero(xs))
        ballot_prefixes_negative(reflect_first_positive_deficit_zero(xs))
    }
}

/// First-bad reflection inverts first-positive reflection before the first positive-leading prefix.
theorem reflect_first_bad_gap_inverts_first_positive_deficit(xs: List[Vote], deficit: Nat) {
    not prefixes_nonpositive_deficit(xs, deficit) implies
        reflect_first_bad_gap(reflect_first_positive_deficit(xs, deficit), deficit) = xs
} by {
    define p(ys: List[Vote]) -> Bool {
        forall(d: Nat) {
            not prefixes_nonpositive_deficit(ys, d) implies
                reflect_first_bad_gap(reflect_first_positive_deficit(ys, d), d) = ys
        }
    }

    p(List.nil[Vote])
    forall(head: Vote, tail: List[Vote]) {
        if p(tail) {
            forall(d: Nat) {
                if not prefixes_nonpositive_deficit(List.cons(head, tail), d) {
                    match head {
                        Vote.positive {
                            if d = Nat.0 {
                                reflect_first_positive_deficit(List.cons(head, tail), d) =
                                    List.cons(Vote.negative, tail)
                                reflect_first_bad_gap(reflect_first_positive_deficit(List.cons(head, tail), d), d) =
                                    List.cons(Vote.positive, tail)
                                reflect_first_bad_gap(reflect_first_positive_deficit(List.cons(head, tail), d), d) =
                                    List.cons(head, tail)
                            }
                            if d != Nat.0 {
                                reflect_first_bad_gap(
                                    reflect_first_positive_deficit(tail, d - Nat.1),
                                    d - Nat.1
                                ) = tail
                                reflect_first_positive_deficit(List.cons(head, tail), d) =
                                    List.cons(Vote.negative, reflect_first_positive_deficit(tail, d - Nat.1))
                                reflect_first_bad_gap(reflect_first_positive_deficit(List.cons(head, tail), d), d) =
                                    List.cons(
                                        Vote.positive,
                                        reflect_first_bad_gap(
                                            reflect_first_positive_deficit(tail, d - Nat.1),
                                            d - Nat.1
                                        )
                                    )
                                reflect_first_bad_gap(reflect_first_positive_deficit(List.cons(head, tail), d), d) =
                                    List.cons(head, tail)
                            }
                            reflect_first_bad_gap(reflect_first_positive_deficit(List.cons(head, tail), d), d) =
                                List.cons(head, tail)
                        }
                        Vote.negative {
                            not prefixes_nonpositive_deficit(tail, d.suc)
                            reflect_first_bad_gap(reflect_first_positive_deficit(tail, d.suc), d.suc) = tail
                            reflect_first_positive_deficit(List.cons(head, tail), d) =
                                List.cons(Vote.positive, reflect_first_positive_deficit(tail, d.suc))
                            reflect_first_bad_gap(reflect_first_positive_deficit(List.cons(head, tail), d), d) =
                                List.cons(
                                    Vote.negative,
                                    reflect_first_bad_gap(reflect_first_positive_deficit(tail, d.suc), d.suc)
                                )
                            reflect_first_bad_gap(reflect_first_positive_deficit(List.cons(head, tail), d), d) =
                                List.cons(head, tail)
                        }
                    }
                    reflect_first_bad_gap(reflect_first_positive_deficit(List.cons(head, tail), d), d) =
                        List.cons(head, tail)
                }
            }
            p(List.cons(head, tail))
        }
    }

    p(xs)
}

/// Zero-gap first-bad reflection inverts zero-deficit first-positive reflection.
theorem reflect_first_bad_gap_zero_inverts_first_positive_deficit_zero(xs: List[Vote]) {
    not prefixes_nonpositive_deficit(xs, Nat.0) implies
        reflect_first_bad_gap_zero(reflect_first_positive_deficit_zero(xs)) = xs
} by {
    if not prefixes_nonpositive_deficit(xs, Nat.0) {
        reflect_first_bad_gap_inverts_first_positive_deficit(xs, Nat.0)
        reflect_first_bad_gap_zero(reflect_first_positive_deficit_zero(xs)) = xs
    }
}

/// Every shifted generated word appears in the reflected bad-word image when positives dominate.
theorem ballot_words_contained_by_reflected_bad_ballot_words(
    positives: Nat,
    negatives: Nat,
    word: List[Vote]
) {
    negatives <= positives and ballot_words(positives.suc, negatives).contains(word) implies
        reflected_bad_ballot_words(positives, negatives).contains(word)
} by {
    if negatives <= positives and ballot_words(positives.suc, negatives).contains(word) {
        ballot_words_sound(positives.suc, negatives, word)
        ballot_word_counts_imp_not_prefixes_nonpositive_deficit_zero(word, positives, negatives)
        let source: List[Vote] = reflect_first_positive_deficit_zero(word)
        reflect_first_positive_deficit_zero_count_shift(word)
        positive_count(word) = positives.suc
        positive_count(source) = positives
        negative_count(word) = negatives
        negative_count(source) = negatives.suc
        ballot_word_has_counts(source, positives, negatives.suc)
        ballot_words_complete(positives, negatives.suc, source)
        reflect_first_positive_deficit_zero_prefixes_negative(word)
        filter_contains_and[List[Vote]](
            ballot_words(positives, negatives.suc),
            ballot_prefixes_negative,
            source
        )
        map_contains_of_contains[List[Vote], List[Vote]](
            bad_ballot_words(positives, negatives.suc),
            reflect_first_bad_gap_zero,
            source
        )
        map(bad_ballot_words(positives, negatives.suc), reflect_first_bad_gap_zero).contains(
            reflect_first_bad_gap_zero(source)
        )
        reflect_first_bad_gap_zero_inverts_first_positive_deficit_zero(word)
        reflected_bad_ballot_words(positives, negatives).contains(word)
    }
}

/// First-positive reflection inverts first-bad reflection before the first negative-leading prefix.
theorem reflect_first_positive_deficit_inverts_first_bad_gap(xs: List[Vote], gap: Nat) {
    not prefixes_nonnegative_gap(xs, gap) implies
        reflect_first_positive_deficit(reflect_first_bad_gap(xs, gap), gap) = xs
} by {
    define p(ys: List[Vote]) -> Bool {
        forall(g: Nat) {
            not prefixes_nonnegative_gap(ys, g) implies
                reflect_first_positive_deficit(reflect_first_bad_gap(ys, g), g) = ys
        }
    }

    p(List.nil[Vote])
    forall(head: Vote, tail: List[Vote]) {
        if p(tail) {
            forall(g: Nat) {
                if not prefixes_nonnegative_gap(List.cons(head, tail), g) {
                    match head {
                        Vote.positive {
                            not prefixes_nonnegative_gap(tail, g.suc)
                            reflect_first_positive_deficit(reflect_first_bad_gap(tail, g.suc), g.suc) = tail
                            reflect_first_bad_gap(List.cons(head, tail), g) =
                                List.cons(Vote.negative, reflect_first_bad_gap(tail, g.suc))
                            reflect_first_positive_deficit(reflect_first_bad_gap(List.cons(head, tail), g), g) =
                                List.cons(
                                    Vote.positive,
                                    reflect_first_positive_deficit(reflect_first_bad_gap(tail, g.suc), g.suc)
                                )
                            reflect_first_positive_deficit(reflect_first_bad_gap(List.cons(head, tail), g), g) =
                                List.cons(head, tail)
                        }
                        Vote.negative {
                            if g = Nat.0 {
                                reflect_first_bad_gap(List.cons(head, tail), g) = List.cons(Vote.positive, tail)
                                reflect_first_positive_deficit(reflect_first_bad_gap(List.cons(head, tail), g), g) =
                                    List.cons(Vote.negative, tail)
                                reflect_first_positive_deficit(reflect_first_bad_gap(List.cons(head, tail), g), g) =
                                    List.cons(head, tail)
                            }
                            if g != Nat.0 {
                                reflect_first_positive_deficit(
                                    reflect_first_bad_gap(tail, g - Nat.1),
                                    g - Nat.1
                                ) = tail
                                reflect_first_bad_gap(List.cons(head, tail), g) =
                                    List.cons(Vote.positive, reflect_first_bad_gap(tail, g - Nat.1))
                                reflect_first_positive_deficit(reflect_first_bad_gap(List.cons(head, tail), g), g) =
                                    List.cons(
                                        Vote.negative,
                                        reflect_first_positive_deficit(
                                            reflect_first_bad_gap(tail, g - Nat.1),
                                            g - Nat.1
                                        )
                                    )
                                reflect_first_positive_deficit(reflect_first_bad_gap(List.cons(head, tail), g), g) =
                                    List.cons(head, tail)
                            }
                            reflect_first_positive_deficit(reflect_first_bad_gap(List.cons(head, tail), g), g) =
                                List.cons(head, tail)
                        }
                    }
                    reflect_first_positive_deficit(reflect_first_bad_gap(List.cons(head, tail), g), g) =
                        List.cons(head, tail)
                }
            }
            p(List.cons(head, tail))
        }
    }

    p(xs)
}

/// Zero-deficit first-positive reflection inverts zero-gap first-bad reflection on bad words.
theorem reflect_first_positive_deficit_zero_inverts_first_bad_gap_zero(xs: List[Vote]) {
    not prefixes_nonnegative_gap(xs, Nat.0) implies
        reflect_first_positive_deficit_zero(reflect_first_bad_gap_zero(xs)) = xs
} by {
    if not prefixes_nonnegative_gap(xs, Nat.0) {
        reflect_first_positive_deficit_inverts_first_bad_gap(xs, Nat.0)
        reflect_first_positive_deficit_zero(reflect_first_bad_gap_zero(xs)) = xs
    }
}

/// Zero-gap first-bad reflection is injective on words with a bad zero-gap prefix.
theorem reflect_first_bad_gap_zero_injective_on_bad(xs: List[Vote], ys: List[Vote]) {
    not prefixes_nonnegative_gap(xs, Nat.0) and not prefixes_nonnegative_gap(ys, Nat.0) and
        reflect_first_bad_gap_zero(xs) = reflect_first_bad_gap_zero(ys) implies xs = ys
} by {
    if not prefixes_nonnegative_gap(xs, Nat.0) and not prefixes_nonnegative_gap(ys, Nat.0) and
        reflect_first_bad_gap_zero(xs) = reflect_first_bad_gap_zero(ys) {
        reflect_first_positive_deficit_zero_inverts_first_bad_gap_zero(xs)
        reflect_first_positive_deficit_zero_inverts_first_bad_gap_zero(ys)
        xs = ys
    }
}

/// Mapping zero-gap first-bad reflection over unique bad words preserves uniqueness.
theorem reflect_first_bad_gap_zero_map_bad_unique(words: List[List[Vote]]) {
    words.is_unique and
        (forall(word: List[Vote]) { words.contains(word) implies not prefixes_nonnegative_gap(word, Nat.0) })
    implies map(words, reflect_first_bad_gap_zero).is_unique
} by {
    define p(items: List[List[Vote]]) -> Bool {
        items.is_unique and
            (forall(word: List[Vote]) { items.contains(word) implies not prefixes_nonnegative_gap(word, Nat.0) })
        implies map(items, reflect_first_bad_gap_zero).is_unique
    }

    if List.nil[List[Vote]].is_unique and
        (forall(word: List[Vote]) {
            List.nil[List[Vote]].contains(word) implies not prefixes_nonnegative_gap(word, Nat.0)
        }) {
        map(List.nil[List[Vote]], reflect_first_bad_gap_zero).is_unique
    }
    p(List.nil[List[Vote]])
    forall(head: List[Vote], tail: List[List[Vote]]) {
        if p(tail) {
            if List.cons(head, tail).is_unique and
                (forall(word: List[Vote]) {
                    List.cons(head, tail).contains(word) implies not prefixes_nonnegative_gap(word, Nat.0)
                }) {
                unique_implies_tail_unique[List[Vote]](head, tail)
                tail.is_unique
                forall(word: List[Vote]) {
                    if tail.contains(word) {
                        List.cons(head, tail).contains(word)
                        not prefixes_nonnegative_gap(word, Nat.0)
                    }
                }
                map(tail, reflect_first_bad_gap_zero).is_unique
                unique_cons_not_contains[List[Vote]](head, tail)
                List.cons(head, tail).contains(head)
                if map(tail, reflect_first_bad_gap_zero).contains(reflect_first_bad_gap_zero(head)) {
                    map_contains[List[Vote], List[Vote]](
                        tail,
                        reflect_first_bad_gap_zero,
                        reflect_first_bad_gap_zero(head)
                    )
                    let other: List[Vote] satisfy {
                        tail.contains(other) and reflect_first_bad_gap_zero(other) = reflect_first_bad_gap_zero(head)
                    }
                    not prefixes_nonnegative_gap(other, Nat.0)
                    reflect_first_bad_gap_zero_injective_on_bad(other, head)
                    other = head
                    false
                }
                cons_unique_of_tail_unique_not_contains[List[Vote]](
                    reflect_first_bad_gap_zero(head),
                    map(tail, reflect_first_bad_gap_zero)
                )
                map(List.cons(head, tail), reflect_first_bad_gap_zero).is_unique
            }
            p(List.cons(head, tail))
        }
    }

    p(words)
}

/// The reflected bad-word image has no duplicate words.
theorem reflected_bad_ballot_words_unique(positives: Nat, negatives: Nat) {
    reflected_bad_ballot_words(positives, negatives).is_unique
} by {
    bad_ballot_words_unique(positives, negatives.suc)
    forall(word: List[Vote]) {
        if bad_ballot_words(positives, negatives.suc).contains(word) {
            filter_contained_by_and[List[Vote]](
                ballot_words(positives, negatives.suc),
                ballot_prefixes_negative,
                word
            )
            ballot_prefixes_negative_imp_not_gap_zero(word)
            not prefixes_nonnegative_gap(word, Nat.0)
        }
    }
    reflect_first_bad_gap_zero_map_bad_unique(bad_ballot_words(positives, negatives.suc))
}

/// The reflected bad-word image has exactly the shifted generated ballot words.
theorem reflected_bad_ballot_words_same_contains(
    positives: Nat,
    negatives: Nat,
    word: List[Vote]
) {
    negatives <= positives implies (
        reflected_bad_ballot_words(positives, negatives).contains(word) =
            ballot_words(positives.suc, negatives).contains(word)
    )
} by {
    if negatives <= positives {
        if reflected_bad_ballot_words(positives, negatives).contains(word) {
            reflected_bad_ballot_words_contained_by_ballot_words(positives, negatives, word)
            ballot_words(positives.suc, negatives).contains(word)
        }
        if ballot_words(positives.suc, negatives).contains(word) {
            ballot_words_contained_by_reflected_bad_ballot_words(positives, negatives, word)
            reflected_bad_ballot_words(positives, negatives).contains(word)
        }
        reflected_bad_ballot_words(positives, negatives).contains(word) =
            ballot_words(positives.suc, negatives).contains(word)
    }
}

/// Every word in the reflected bad-word image has the shifted vote counts.
theorem reflected_bad_ballot_words_sound_counts(positives: Nat, negatives: Nat, word: List[Vote]) {
    reflected_bad_ballot_words(positives, negatives).contains(word) implies
        ballot_word_has_counts(word, positives.suc, negatives)
} by {
    if reflected_bad_ballot_words(positives, negatives).contains(word) {
        reflected_bad_ballot_words_contained_by_ballot_words(positives, negatives, word)
        ballot_words_sound(positives.suc, negatives, word)
        ballot_word_has_counts(word, positives.suc, negatives)
    }
}

/// Every word with shifted counts lies in the reflected bad-word image when positives dominate.
theorem reflected_bad_ballot_words_complete_counts(positives: Nat, negatives: Nat, word: List[Vote]) {
    negatives <= positives and ballot_word_has_counts(word, positives.suc, negatives) implies
        reflected_bad_ballot_words(positives, negatives).contains(word)
} by {
    if negatives <= positives and ballot_word_has_counts(word, positives.suc, negatives) {
        ballot_words_complete(positives.suc, negatives, word)
        ballot_words_contained_by_reflected_bad_ballot_words(positives, negatives, word)
        reflected_bad_ballot_words(positives, negatives).contains(word)
    }
}

/// Reflected bad-word membership is exactly the shifted count predicate when positives dominate.
theorem reflected_bad_ballot_words_contains_iff_counts(positives: Nat, negatives: Nat, word: List[Vote]) {
    negatives <= positives implies
        reflected_bad_ballot_words(positives, negatives).contains(word) =
            ballot_word_has_counts(word, positives.suc, negatives)
} by {
    if negatives <= positives {
        if reflected_bad_ballot_words(positives, negatives).contains(word) {
            reflected_bad_ballot_words_sound_counts(positives, negatives, word)
            ballot_word_has_counts(word, positives.suc, negatives)
        }
        if ballot_word_has_counts(word, positives.suc, negatives) {
            reflected_bad_ballot_words_complete_counts(positives, negatives, word)
            reflected_bad_ballot_words(positives, negatives).contains(word)
        }
    }
}

/// Bad ballot words with one extra negative vote are counted by shifted generated words.
theorem bad_ballot_words_suc_length_eq_ballot_words_shifted(positives: Nat, negatives: Nat) {
    negatives <= positives implies
        bad_ballot_words(positives, negatives.suc).length = ballot_words(positives.suc, negatives).length
} by {
    if negatives <= positives {
        reflected_bad_ballot_words_unique(positives, negatives)
        ballot_words_unique(positives.suc, negatives)
        forall(word: List[Vote]) {
            reflected_bad_ballot_words_same_contains(positives, negatives, word)
            reflected_bad_ballot_words(positives, negatives).contains(word) =
                ballot_words(positives.suc, negatives).contains(word)
        }
        unique_same_contains_imp_permutation[List[Vote]](
            reflected_bad_ballot_words(positives, negatives),
            ballot_words(positives.suc, negatives)
        )
        permutation_preserves_length[List[Vote]](
            reflected_bad_ballot_words(positives, negatives),
            ballot_words(positives.suc, negatives)
        )
        reflected_bad_ballot_words(positives, negatives).length = ballot_words(positives.suc, negatives).length
        map_length[List[Vote], List[Vote]](bad_ballot_words(positives, negatives.suc), reflect_first_bad_gap_zero)
        bad_ballot_words(positives, negatives.suc).length = ballot_words(positives.suc, negatives).length
    }
}

/// Weak ballot words are counted by a binomial difference.
theorem weak_ballot_words_suc_length_binom_difference(positives: Nat, negatives: Nat) {
    negatives <= positives implies
        weak_ballot_words(positives, negatives.suc).length +
            (positives + negatives.suc).binom(positives.suc) =
            (positives + negatives.suc).binom(positives)
} by {
    if negatives <= positives {
        weak_bad_ballot_words_partition_length(positives, negatives.suc)
        bad_ballot_words_suc_length_eq_ballot_words_shifted(positives, negatives)
        ballot_words_length_binom(positives, negatives.suc)
        ballot_words_length_binom(positives.suc, negatives)
        weak_ballot_words(positives, negatives.suc).length +
            (positives + negatives.suc).binom(positives.suc) =
            (positives + negatives.suc).binom(positives)
    }
}

/// Strict successful ballot words are counted by the same binomial difference after removing the first vote.
theorem successful_ballot_words_suc_suc_length_binom_difference(positives: Nat, negatives: Nat) {
    negatives <= positives implies
        successful_ballot_words(positives.suc, negatives.suc).length +
            (positives + negatives.suc).binom(positives.suc) =
            (positives + negatives.suc).binom(positives)
} by {
    if negatives <= positives {
        successful_ballot_words_suc_length_eq_weak(positives, negatives.suc)
        weak_ballot_words_suc_length_binom_difference(positives, negatives)
        successful_ballot_words(positives.suc, negatives.suc).length +
            (positives + negatives.suc).binom(positives.suc) =
            (positives + negatives.suc).binom(positives)
    }
}
