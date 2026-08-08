from algebra.add_comm_group import AddCommGroup
from affine_space import AffineSpace, affine_zero_vadd, affine_add_vadd,
    affine_vsub_vadd, affine_vadd_vsub, affine_vsub_self,
    affine_space_product, affine_space_product_vadd_apply,
    affine_space_product_vsub_apply
from list import List, add_assoc, add_contains_left, add_contains_or, fold_right,
    add_contains_right, is_permutation, list_contains_implies_count_geq_one,
    list_not_contains_impl_count_zero, unique_preserves_contains
from data.basic.logic import and_assoc, and_comm, and_or_distrib_left, and_self,
    not_and, not_exists_forward, or_and_distrib_left, or_assoc, or_comm, or_self
from pair import Pair, pair_assoc_left, pair_assoc_left_first,
    pair_assoc_left_right, pair_assoc_left_second, pair_assoc_right,
    pair_ext, pair_new_first, pair_new_second, swap_first, swap_second

/// True if a subset of an affine space is closed under the operation
/// `(p, q, r) |-> vadd(vsub(p, q), r)`, the affine combination
/// `p - q + r`.
define affine_subspace_constraint[V: AddCommGroup, P](
    a: AffineSpace[V, P], contains: P -> Bool) -> Bool {
    forall(p: P, q: P, r: P) {
        contains(p) and contains(q) and contains(r)
            implies contains(a.vadd(a.vsub(p, q), r))
    }
}

/// An affine subspace of an affine space `P` over `V`, represented as a
/// subset closed under the affine combination `p - q + r`. This subset
/// may be empty.
structure AffineSubspace[V: AddCommGroup, P] {
    /// The ambient affine space.
    space: AffineSpace[V, P]
    /// True if the given point is a member of this affine subspace.
    contains: P -> Bool
} constraint {
    affine_subspace_constraint(space, contains)
}

/// Membership in an affine subspace constructed from a specified predicate.
theorem affine_subspace_new_contains_apply[V: AddCommGroup, P](
    a: AffineSpace[V, P],
    contains: P -> Bool,
    s: AffineSubspace[V, P],
    p: P
) {
    AffineSubspace[V, P].new(a, contains) = Option.some(s) implies
        s.contains(p) = contains(p)
} by {
    if AffineSubspace[V, P].new(a, contains) = Option.some(s) {
        s.contains(p) = contains(p)
    }
}

/// Affine subspace extensionality.
theorem affine_subspace_ext[V: AddCommGroup, P](
    s: AffineSubspace[V, P],
    t: AffineSubspace[V, P]) {
    s.space = t.space and (forall(x: P) { s.contains(x) = t.contains(x) })
        implies s = t
} by {
    if s.space = t.space and forall(x: P) { s.contains(x) = t.contains(x) } {
        s.contains = t.contains
    }
}

/// The membership predicate of an affine subspace satisfies the
/// affine-combination closure property.
theorem affine_subspace_closed[V: AddCommGroup, P](
    s: AffineSubspace[V, P],
    p: P, q: P, r: P) {
    s.contains(p) and s.contains(q) and s.contains(r)
        implies s.contains(s.space.vadd(s.space.vsub(p, q), r))
} by {
    affine_subspace_constraint(s.space, s.contains) = forall(x: P, y: P, z: P) {
        s.contains(x) and s.contains(y) and s.contains(z)
            implies s.contains(s.space.vadd(s.space.vsub(x, y), z))
    }
}

/// The everywhere-false predicate.
let affine_subspace_empty_contains[P]: P -> Bool = function(p: P) {
    false
}

/// The everywhere-true predicate.
let affine_subspace_univ_contains[P]: P -> Bool = function(p: P) {
    true
}

/// The empty subset satisfies the affine-subspace closure property.
theorem affine_subspace_empty_constraint[V: AddCommGroup, P](
    a: AffineSpace[V, P]) {
    affine_subspace_constraint(a, affine_subspace_empty_contains[P])
} by {
    forall(p: P, q: P, r: P) {
        affine_subspace_empty_contains[P](p) = false
    }
}

/// The full subset satisfies the affine-subspace closure property.
theorem affine_subspace_univ_constraint[V: AddCommGroup, P](
    a: AffineSpace[V, P]) {
    affine_subspace_constraint(a, affine_subspace_univ_contains[P])
} by {
    forall(p: P, q: P, r: P) {
        affine_subspace_univ_contains[P](a.vadd(a.vsub(p, q), r)) = true
    }
}

/// The empty affine subspace of an affine space.
let affine_subspace_empty[V: AddCommGroup, P](a: AffineSpace[V, P]) -> result: AffineSubspace[V, P] satisfy {
    AffineSubspace[V, P].new(a, affine_subspace_empty_contains[P]) =
        Option.some(result)
} by {
    affine_subspace_empty_constraint(a)
}

/// The full affine subspace consisting of all points of an affine space.
let affine_subspace_univ[V: AddCommGroup, P](a: AffineSpace[V, P]) -> result: AffineSubspace[V, P] satisfy {
    AffineSubspace[V, P].new(a, affine_subspace_univ_contains[P]) =
        Option.some(result)
} by {
    affine_subspace_univ_constraint(a)
}

/// The ambient affine space of the full affine subspace.
theorem affine_subspace_univ_space[V: AddCommGroup, P](
    a: AffineSpace[V, P]
) {
    affine_subspace_univ(a).space = a
}

/// The empty affine subspace contains no points.
theorem affine_subspace_empty_not_contains[V: AddCommGroup, P](
    a: AffineSpace[V, P], p: P) {
    not affine_subspace_empty(a).contains(p)
} by {
    affine_subspace_empty(a).contains = affine_subspace_empty_contains[P]
    affine_subspace_empty_contains[P](p) = false
}

/// The full affine subspace contains every point.
theorem affine_subspace_univ_contains_all[V: AddCommGroup, P](
    a: AffineSpace[V, P], p: P) {
    affine_subspace_univ(a).contains(p)
} by {
    affine_subspace_univ(a).contains = affine_subspace_univ_contains[P]
}

/// True if an affine subspace contains a point.
define affine_subspace_nonempty[V: AddCommGroup, P](
    a: AffineSubspace[V, P]
) -> Bool {
    exists(p: P) {
        a.contains(p)
    }
}

/// An affine subspace containing a specified point is nonempty.
theorem affine_subspace_nonempty_of_contains[V: AddCommGroup, P](
    a: AffineSubspace[V, P],
    p: P
) {
    a.contains(p) implies affine_subspace_nonempty(a)
} by {
    if a.contains(p) {
        exists(q: P) {
            a.contains(q)
        }
        affine_subspace_nonempty(a)
    }
}

/// An affine subspace over a specified ambient space is the empty affine
/// subspace exactly when it is not nonempty.
theorem affine_subspace_eq_empty_iff_not_nonempty[V: AddCommGroup, P](
    a: AffineSubspace[V, P],
    space: AffineSpace[V, P]
) {
    a.space = space implies
        ((a = affine_subspace_empty(space)) =
            not affine_subspace_nonempty(a))
} by {
    if a.space = space {
        let empty = affine_subspace_empty(space)
        if a = empty {
            if affine_subspace_nonempty(a) {
                affine_subspace_nonempty(a) = exists(p: P) {
                    a.contains(p)
                }
                let p: P satisfy {
                    a.contains(p)
                }
                affine_subspace_empty_not_contains(space, p)
                false
            }
            not affine_subspace_nonempty(a)
        }
        if not affine_subspace_nonempty(a) {
            affine_subspace_nonempty(a) = exists(p: P) {
                a.contains(p)
            }
            not exists(p: P) {
                a.contains(p)
            }
            not_exists_forward(a.contains)
            forall(p: P) {
                not a.contains(p)
            }
            forall(p: P) {
                affine_subspace_empty_not_contains(space, p)
                not empty.contains(p)
                a.contains(p) = empty.contains(p)
            }
            AffineSubspace[V, P].new(
                space, affine_subspace_empty_contains[P]) =
                Option.some(empty)
            empty.space = space
            affine_subspace_ext(a, empty)
            a = empty
        }
        (a = affine_subspace_empty(space)) =
            not affine_subspace_nonempty(a)
    }
}

/// Coordinatewise membership in the product of two affine subspaces.
define affine_subspace_product_contains[V: AddCommGroup, W: AddCommGroup, P, Q](
    a: AffineSubspace[V, P],
    b: AffineSubspace[W, Q],
    p: Pair[P, Q]
) -> Bool {
    a.contains(p.first) and b.contains(p.second)
}

/// Coordinatewise product membership is closed under affine combinations in
/// the product affine space.
theorem affine_subspace_product_constraint[V: AddCommGroup, W: AddCommGroup, P, Q](
    a: AffineSubspace[V, P],
    b: AffineSubspace[W, Q]
) {
    affine_subspace_constraint(
        affine_space_product(a.space, b.space),
        affine_subspace_product_contains(a, b))
} by {
    let product_space = affine_space_product(a.space, b.space)
    affine_subspace_constraint(
        product_space, affine_subspace_product_contains(a, b)) =
        forall(p: Pair[P, Q], q: Pair[P, Q], r: Pair[P, Q]) {
            affine_subspace_product_contains(a, b, p)
            and affine_subspace_product_contains(a, b, q)
            and affine_subspace_product_contains(a, b, r) implies
                affine_subspace_product_contains(
                    a, b,
                    product_space.vadd(product_space.vsub(p, q), r))
        }
    forall(p: Pair[P, Q], q: Pair[P, Q], r: Pair[P, Q]) {
        if affine_subspace_product_contains(a, b, p)
            and affine_subspace_product_contains(a, b, q)
            and affine_subspace_product_contains(a, b, r) {
            affine_subspace_product_contains(a, b, p) =
                (a.contains(p.first) and b.contains(p.second))
            affine_subspace_product_contains(a, b, q) =
                (a.contains(q.first) and b.contains(q.second))
            affine_subspace_product_contains(a, b, r) =
                (a.contains(r.first) and b.contains(r.second))

            affine_subspace_closed(a, p.first, q.first, r.first)
            affine_subspace_closed(b, p.second, q.second, r.second)

            affine_space_product_vsub_apply(a.space, b.space, p, q)
            let difference = product_space.vsub(p, q)
            difference = Pair.new(
                a.space.vsub(p.first, q.first),
                b.space.vsub(p.second, q.second))
            pair_new_first(
                a.space.vsub(p.first, q.first),
                b.space.vsub(p.second, q.second))
            pair_new_second(
                a.space.vsub(p.first, q.first),
                b.space.vsub(p.second, q.second))
            difference.first = a.space.vsub(p.first, q.first)
            difference.second = b.space.vsub(p.second, q.second)

            affine_space_product_vadd_apply(
                a.space, b.space, difference, r)
            let translated = product_space.vadd(difference, r)
            translated = Pair.new(
                a.space.vadd(difference.first, r.first),
                b.space.vadd(difference.second, r.second))
            pair_new_first(
                a.space.vadd(difference.first, r.first),
                b.space.vadd(difference.second, r.second))
            pair_new_second(
                a.space.vadd(difference.first, r.first),
                b.space.vadd(difference.second, r.second))
            translated.first =
                a.space.vadd(a.space.vsub(p.first, q.first), r.first)
            translated.second =
                b.space.vadd(b.space.vsub(p.second, q.second), r.second)
            a.contains(translated.first)
            b.contains(translated.second)
            affine_subspace_product_contains(a, b, translated)
            affine_subspace_product_contains(
                a, b,
                product_space.vadd(product_space.vsub(p, q), r))
        }
    }
    affine_subspace_constraint(
        product_space, affine_subspace_product_contains(a, b))
}

/// The product of two affine subspaces, with membership defined
/// coordinatewise.
let affine_subspace_product[V: AddCommGroup, W: AddCommGroup, P, Q](a: AffineSubspace[V, P], b: AffineSubspace[W, Q]) -> result: AffineSubspace[Pair[V, W], Pair[P, Q]] satisfy {
    AffineSubspace[Pair[V, W], Pair[P, Q]].new(
        affine_space_product(a.space, b.space),
        affine_subspace_product_contains(a, b)) = Option.some(result)
} by {
    affine_subspace_product_constraint(a, b)
}

/// The ambient affine space of a product affine subspace is the product of
/// the two ambient affine spaces.
theorem affine_subspace_product_space[V: AddCommGroup, W: AddCommGroup, P, Q](
    a: AffineSubspace[V, P],
    b: AffineSubspace[W, Q]
) {
    affine_subspace_product(a, b).space =
        affine_space_product(a.space, b.space)
}

/// Membership in a product affine subspace is membership in both coordinates.
theorem affine_subspace_product_contains_iff[V: AddCommGroup, W: AddCommGroup, P, Q](
    a: AffineSubspace[V, P],
    b: AffineSubspace[W, Q],
    p: Pair[P, Q]
) {
    affine_subspace_product(a, b).contains(p) =
        (a.contains(p.first) and b.contains(p.second))
} by {
    AffineSubspace[Pair[V, W], Pair[P, Q]].new(
        affine_space_product(a.space, b.space),
        affine_subspace_product_contains(a, b)) =
        Option.some(affine_subspace_product(a, b))
    affine_subspace_new_contains_apply(
        affine_space_product(a.space, b.space),
        affine_subspace_product_contains(a, b),
        affine_subspace_product(a, b),
        p)
    affine_subspace_product(a, b).contains(p) =
        affine_subspace_product_contains(a, b, p)
    affine_subspace_product_contains(a, b, p) =
        (a.contains(p.first) and b.contains(p.second))
}

/// Membership in a product with a full right factor is membership in the
/// left coordinate.
theorem affine_subspace_product_univ_right_contains_iff[V: AddCommGroup, W: AddCommGroup, P, Q](
    a: AffineSubspace[V, P],
    b: AffineSpace[W, Q],
    p: Pair[P, Q]
) {
    affine_subspace_product(a, affine_subspace_univ(b)).contains(p) =
        a.contains(p.first)
} by {
    affine_subspace_product_contains_iff(a, affine_subspace_univ(b), p)
    affine_subspace_univ_contains_all(b, p.second)
}

/// If `y` is the left coordinate, membership in a product with a full right
/// factor is membership of `y` in the left subspace.
theorem affine_subspace_product_univ_right_contains_of_eq[V: AddCommGroup, W: AddCommGroup, P, Q](
    a: AffineSubspace[V, P],
    b: AffineSpace[W, Q],
    p: Pair[P, Q],
    y: P
) {
    y = p.first implies
        affine_subspace_product(a, affine_subspace_univ(b)).contains(p) =
            a.contains(y)
} by {
    if y = p.first {
        affine_subspace_product_univ_right_contains_iff(a, b, p)
    }
}

/// Membership in a product with a full left factor is membership in the
/// right coordinate.
theorem affine_subspace_product_univ_left_contains_iff[V: AddCommGroup, W: AddCommGroup, P, Q](
    a: AffineSpace[V, P],
    b: AffineSubspace[W, Q],
    p: Pair[P, Q]
) {
    affine_subspace_product(affine_subspace_univ(a), b).contains(p) =
        b.contains(p.second)
} by {
    affine_subspace_product_contains_iff(affine_subspace_univ(a), b, p)
    affine_subspace_univ_contains_all(a, p.first)
}

/// If `y` is the right coordinate, membership in a product with a full left
/// factor is membership of `y` in the right subspace.
theorem affine_subspace_product_univ_left_contains_of_eq[V: AddCommGroup, W: AddCommGroup, P, Q](
    a: AffineSpace[V, P],
    b: AffineSubspace[W, Q],
    p: Pair[P, Q],
    y: Q
) {
    y = p.second implies
        affine_subspace_product(affine_subspace_univ(a), b).contains(p) =
            b.contains(y)
} by {
    if y = p.second {
        affine_subspace_product_univ_left_contains_iff(a, b, p)
    }
}

/// A pair belongs to a product affine subspace exactly when each specified
/// coordinate belongs to its corresponding factor.
theorem affine_subspace_product_contains_pair_iff[V: AddCommGroup, W: AddCommGroup, P, Q](
    a: AffineSubspace[V, P],
    b: AffineSubspace[W, Q],
    p: P,
    q: Q
) {
    affine_subspace_product(a, b).contains(Pair.new(p, q)) =
        (a.contains(p) and b.contains(q))
} by {
    let pair = Pair.new(p, q)
    pair_new_first(p, q)
    pair_new_second(p, q)
    pair.first = p
    pair.second = q
    affine_subspace_product_contains_iff(a, b, pair)
}

/// Swapping the coordinates of a point exchanges the factors of product
/// affine-subspace membership.
theorem affine_subspace_product_contains_swap_iff[V: AddCommGroup, W: AddCommGroup, P, Q](
    a: AffineSubspace[V, P],
    b: AffineSubspace[W, Q],
    p: Pair[P, Q]
) {
    affine_subspace_product(b, a).contains(p.swap) =
        affine_subspace_product(a, b).contains(p)
} by {
    affine_subspace_product_contains_iff(b, a, p.swap)
    swap_first(p)
    swap_second(p)
    affine_subspace_product(b, a).contains(p.swap) =
        (b.contains(p.second) and a.contains(p.first))

    affine_subspace_product_contains_iff(a, b, p)
    affine_subspace_product(a, b).contains(p) =
        (a.contains(p.first) and b.contains(p.second))
    and_comm(a.contains(p.first), b.contains(p.second))
    affine_subspace_product(b, a).contains(p.swap) =
        affine_subspace_product(a, b).contains(p)
}

/// Left association of nested coordinates transports membership between the
/// two associations of a triple product affine subspace.
theorem affine_subspace_product_contains_assoc_left_iff[V: AddCommGroup, W: AddCommGroup, X: AddCommGroup, P, Q, R](
    a: AffineSubspace[V, P],
    b: AffineSubspace[W, Q],
    c: AffineSubspace[X, R],
    p: Pair[P, Pair[Q, R]]
) {
    affine_subspace_product(
        affine_subspace_product(a, b), c).contains(pair_assoc_left(p)) =
        affine_subspace_product(
            a, affine_subspace_product(b, c)).contains(p)
} by {
    let left = affine_subspace_product(
        affine_subspace_product(a, b), c)
    let right = affine_subspace_product(
        a, affine_subspace_product(b, c))
    let associated = pair_assoc_left(p)
    let pq = Pair.new(p.first, p.second.first)
    let a_contains = a.contains(p.first)
    let b_contains = b.contains(p.second.first)
    let c_contains = c.contains(p.second.second)

    pair_assoc_left_first(p)
    associated.first = pq
    pair_assoc_left_second(p)
    associated.second = p.second.second
    affine_subspace_product_contains_iff(
        affine_subspace_product(a, b), c, associated)
    affine_subspace_product_contains_pair_iff(
        a, b, p.first, p.second.first)
    affine_subspace_product(a, b).contains(pq) =
        (a_contains and b_contains)
    left.contains(associated) =
        ((a_contains and b_contains) and c_contains)

    affine_subspace_product_contains_iff(
        a, affine_subspace_product(b, c), p)
    affine_subspace_product_contains_iff(b, c, p.second)
    affine_subspace_product(b, c).contains(p.second) =
        (b_contains and c_contains)
    right.contains(p) =
        (a_contains and (b_contains and c_contains))

    and_assoc(a_contains, b_contains, c_contains)
    left.contains(associated) = right.contains(p)
    affine_subspace_product(
        affine_subspace_product(a, b), c).contains(pair_assoc_left(p)) =
        affine_subspace_product(
            a, affine_subspace_product(b, c)).contains(p)
}

/// Right association of nested coordinates transports membership between the
/// two associations of a triple product affine subspace.
theorem affine_subspace_product_contains_assoc_right_iff[V: AddCommGroup, W: AddCommGroup, X: AddCommGroup, P, Q, R](
    a: AffineSubspace[V, P],
    b: AffineSubspace[W, Q],
    c: AffineSubspace[X, R],
    p: Pair[Pair[P, Q], R]
) {
    affine_subspace_product(
        a, affine_subspace_product(b, c)).contains(pair_assoc_right(p)) =
        affine_subspace_product(
            affine_subspace_product(a, b), c).contains(p)
} by {
    let associated = pair_assoc_right(p)
    affine_subspace_product_contains_assoc_left_iff(
        a, b, c, associated)
    pair_assoc_left_right(p)
    pair_assoc_left(associated) = p
    affine_subspace_product(
        a, affine_subspace_product(b, c)).contains(associated) =
        affine_subspace_product(
            affine_subspace_product(a, b), c).contains(p)
    affine_subspace_product(
        a, affine_subspace_product(b, c)).contains(pair_assoc_right(p)) =
        affine_subspace_product(
            affine_subspace_product(a, b), c).contains(p)
}

/// A product affine subspace is nonempty exactly when both factor affine
/// subspaces are nonempty.
theorem affine_subspace_product_nonempty_iff[V: AddCommGroup, W: AddCommGroup, P, Q](
    a: AffineSubspace[V, P],
    b: AffineSubspace[W, Q]
) {
    affine_subspace_nonempty(affine_subspace_product(a, b)) =
        (affine_subspace_nonempty(a) and affine_subspace_nonempty(b))
} by {
    let product_nonempty =
        affine_subspace_nonempty(affine_subspace_product(a, b))
    let factors_nonempty =
        affine_subspace_nonempty(a) and affine_subspace_nonempty(b)
    if product_nonempty {
        affine_subspace_nonempty(affine_subspace_product(a, b)) =
            exists(pair: Pair[P, Q]) {
                affine_subspace_product(a, b).contains(pair)
            }
        let pair: Pair[P, Q] satisfy {
            affine_subspace_product(a, b).contains(pair)
        }
        affine_subspace_product_contains_iff(a, b, pair)
        a.contains(pair.first)
        b.contains(pair.second)
        affine_subspace_nonempty_of_contains(a, pair.first)
        affine_subspace_nonempty(a)
        affine_subspace_nonempty_of_contains(b, pair.second)
        affine_subspace_nonempty(b)
        factors_nonempty
    }
    if factors_nonempty {
        affine_subspace_nonempty(a) = exists(p: P) {
            a.contains(p)
        }
        affine_subspace_nonempty(b) = exists(q: Q) {
            b.contains(q)
        }
        let p: P satisfy {
            a.contains(p)
        }
        let q: Q satisfy {
            b.contains(q)
        }
        affine_subspace_product_contains_pair_iff(a, b, p, q)
        affine_subspace_product(a, b).contains(Pair.new(p, q))
        affine_subspace_nonempty_of_contains(
            affine_subspace_product(a, b), Pair.new(p, q))
        product_nonempty
    }
    product_nonempty = factors_nonempty
    affine_subspace_nonempty(affine_subspace_product(a, b)) =
        (affine_subspace_nonempty(a) and affine_subspace_nonempty(b))
}

/// The common membership predicate of two affine subspaces.
define affine_subspace_intersection_contains[V: AddCommGroup, P](
    a: AffineSubspace[V, P], b: AffineSubspace[V, P], x: P) -> Bool {
    a.contains(x) and b.contains(x)
}

/// The common part of two affine subspaces over the same affine space is closed under
/// the affine combination `p - q + r`.
theorem affine_subspace_intersection_constraint[V: AddCommGroup, P](
    a: AffineSubspace[V, P], b: AffineSubspace[V, P]) {
    a.space = b.space implies
        affine_subspace_constraint(a.space, affine_subspace_intersection_contains(a, b))
} by {
    if a.space = b.space {
        forall(p: P, q: P, r: P) {
            if affine_subspace_intersection_contains(a, b, p)
               and affine_subspace_intersection_contains(a, b, q)
               and affine_subspace_intersection_contains(a, b, r) {
                a.contains(q)
                a.contains(r)
                b.contains(q)
                b.contains(r)
                affine_subspace_closed(a, p, q, r)
                a.contains(a.space.vadd(a.space.vsub(p, q), r))
                affine_subspace_closed(b, p, q, r)
                b.contains(b.space.vadd(b.space.vsub(p, q), r))
                affine_subspace_intersection_contains(a, b,
                    a.space.vadd(a.space.vsub(p, q), r))
            }
        }
    }
}

attributes AffineSubspace[V: AddCommGroup, P] {
    /// Affine subspace extensionality from equality of ambient space and pointwise membership.
    let ext = affine_subspace_ext[V, P]
}

/// True if every point of one affine subspace belongs to another.
define affine_subspace_subset[V: AddCommGroup, P](
    a: AffineSubspace[V, P], b: AffineSubspace[V, P]) -> Bool {
    forall(x: P) {
        a.contains(x) implies b.contains(x)
    }
}

/// Every affine subspace contains itself.
theorem affine_subspace_subset_refl[V: AddCommGroup, P](
    a: AffineSubspace[V, P]) {
    affine_subspace_subset(a, a)
}

/// Affine subspace containment is transitive.
theorem affine_subspace_subset_trans[V: AddCommGroup, P](
    a: AffineSubspace[V, P],
    b: AffineSubspace[V, P],
    c: AffineSubspace[V, P]) {
    affine_subspace_subset(a, b) and affine_subspace_subset(b, c)
        implies affine_subspace_subset(a, c)
} by {
    if affine_subspace_subset(a, b) and affine_subspace_subset(b, c) {
        affine_subspace_subset(a, b) = forall(x: P) {
            a.contains(x) implies b.contains(x)
        }
        affine_subspace_subset(b, c) = forall(x: P) {
            b.contains(x) implies c.contains(x)
        }
        forall(x: P) {
            if a.contains(x) {
                c.contains(x)
            }
        }
    }
}

/// Two affine subspaces over the same ambient space are equal when each is
/// contained in the other.
theorem affine_subspace_subset_antisymm[V: AddCommGroup, P](
    a: AffineSubspace[V, P],
    b: AffineSubspace[V, P]
) {
    a.space = b.space and affine_subspace_subset(a, b)
        and affine_subspace_subset(b, a) implies a = b
} by {
    if a.space = b.space and affine_subspace_subset(a, b)
        and affine_subspace_subset(b, a) {
        affine_subspace_subset(a, b) = forall(x: P) {
            a.contains(x) implies b.contains(x)
        }
        affine_subspace_subset(b, a) = forall(x: P) {
            b.contains(x) implies a.contains(x)
        }
        forall(x: P) {
            a.contains(x) = b.contains(x)
        }
        a.space = b.space and forall(x: P) {
            a.contains(x) = b.contains(x)
        }
        affine_subspace_ext(a, b)
        a = b
    }
}

/// Coordinatewise containment implies containment of product affine
/// subspaces.
theorem affine_subspace_product_subset_of_subset_left_right[V: AddCommGroup, W: AddCommGroup, P, Q](
    a: AffineSubspace[V, P],
    b: AffineSubspace[W, Q],
    c: AffineSubspace[V, P],
    d: AffineSubspace[W, Q]
) {
    affine_subspace_subset(a, c) and affine_subspace_subset(b, d) implies
        affine_subspace_subset(
            affine_subspace_product(a, b),
            affine_subspace_product(c, d))
} by {
    if affine_subspace_subset(a, c) and affine_subspace_subset(b, d) {
        affine_subspace_subset(a, c) = forall(x: P) {
            a.contains(x) implies c.contains(x)
        }
        affine_subspace_subset(b, d) = forall(y: Q) {
            b.contains(y) implies d.contains(y)
        }
        forall(p: Pair[P, Q]) {
            if affine_subspace_product(a, b).contains(p) {
                affine_subspace_product_contains_iff(a, b, p)
                a.contains(p.first)
                b.contains(p.second)
                c.contains(p.first)
                d.contains(p.second)
                affine_subspace_product_contains_iff(c, d, p)
                affine_subspace_product(c, d).contains(p)
            }
        }
    }
}

/// Enlarging the first affine subspace enlarges the product while the second
/// affine subspace remains fixed.
theorem affine_subspace_product_mono_left[V: AddCommGroup, W: AddCommGroup, P, Q](
    a: AffineSubspace[V, P],
    b: AffineSubspace[W, Q],
    c: AffineSubspace[V, P]
) {
    affine_subspace_subset(a, c) implies
        affine_subspace_subset(
            affine_subspace_product(a, b),
            affine_subspace_product(c, b))
} by {
    if affine_subspace_subset(a, c) {
        affine_subspace_subset_refl(b)
        affine_subspace_product_subset_of_subset_left_right(a, b, c, b)
        affine_subspace_subset(
            affine_subspace_product(a, b),
            affine_subspace_product(c, b))
    }
}

/// Enlarging the second affine subspace enlarges the product while the first
/// affine subspace remains fixed.
theorem affine_subspace_product_mono_right[V: AddCommGroup, W: AddCommGroup, P, Q](
    a: AffineSubspace[V, P],
    b: AffineSubspace[W, Q],
    d: AffineSubspace[W, Q]
) {
    affine_subspace_subset(b, d) implies
        affine_subspace_subset(
            affine_subspace_product(a, b),
            affine_subspace_product(a, d))
} by {
    if affine_subspace_subset(b, d) {
        affine_subspace_subset_refl(a)
        affine_subspace_product_subset_of_subset_left_right(a, b, a, d)
        affine_subspace_subset(
            affine_subspace_product(a, b),
            affine_subspace_product(a, d))
    }
}

/// Containment of product affine subspaces implies containment of their first
/// coordinates when the second source factor has a point.
theorem affine_subspace_subset_left_of_product_subset[V: AddCommGroup, W: AddCommGroup, P, Q](
    a: AffineSubspace[V, P],
    b: AffineSubspace[W, Q],
    c: AffineSubspace[V, P],
    d: AffineSubspace[W, Q],
    q: Q
) {
    b.contains(q) and
    affine_subspace_subset(
        affine_subspace_product(a, b),
        affine_subspace_product(c, d)) implies
        affine_subspace_subset(a, c)
} by {
    if b.contains(q) and
        affine_subspace_subset(
            affine_subspace_product(a, b),
            affine_subspace_product(c, d)) {
        affine_subspace_subset(
            affine_subspace_product(a, b),
            affine_subspace_product(c, d)) =
            forall(p: Pair[P, Q]) {
                affine_subspace_product(a, b).contains(p) implies
                    affine_subspace_product(c, d).contains(p)
            }
        forall(x: P) {
            if a.contains(x) {
                let p = Pair.new(x, q)
                pair_new_first(x, q)
                pair_new_second(x, q)
                p.first = x
                p.second = q
                affine_subspace_product_contains_iff(a, b, p)
                affine_subspace_product(a, b).contains(p)
                affine_subspace_product(c, d).contains(p)
                affine_subspace_product_contains_iff(c, d, p)
                c.contains(x)
            }
        }
    }
}

/// Containment of product affine subspaces implies containment of their
/// second coordinates when the first source factor has a point.
theorem affine_subspace_subset_right_of_product_subset[V: AddCommGroup, W: AddCommGroup, P, Q](
    a: AffineSubspace[V, P],
    b: AffineSubspace[W, Q],
    c: AffineSubspace[V, P],
    d: AffineSubspace[W, Q],
    p: P
) {
    a.contains(p) and
    affine_subspace_subset(
        affine_subspace_product(a, b),
        affine_subspace_product(c, d)) implies
        affine_subspace_subset(b, d)
} by {
    if a.contains(p) and
        affine_subspace_subset(
            affine_subspace_product(a, b),
            affine_subspace_product(c, d)) {
        affine_subspace_subset(
            affine_subspace_product(a, b),
            affine_subspace_product(c, d)) =
            forall(q: Pair[P, Q]) {
                affine_subspace_product(a, b).contains(q) implies
                    affine_subspace_product(c, d).contains(q)
            }
        forall(y: Q) {
            if b.contains(y) {
                let q = Pair.new(p, y)
                pair_new_first(p, y)
                pair_new_second(p, y)
                q.first = p
                q.second = y
                affine_subspace_product_contains_iff(a, b, q)
                affine_subspace_product(a, b).contains(q)
                affine_subspace_product(c, d).contains(q)
                affine_subspace_product_contains_iff(c, d, q)
                d.contains(y)
            }
        }
    }
}

/// Containment between products with inhabited source factors is exactly
/// coordinatewise containment.
theorem affine_subspace_product_subset_iff[V: AddCommGroup, W: AddCommGroup, P, Q](
    a: AffineSubspace[V, P],
    b: AffineSubspace[W, Q],
    c: AffineSubspace[V, P],
    d: AffineSubspace[W, Q],
    p: P,
    q: Q
) {
    a.contains(p) and b.contains(q) implies
        (affine_subspace_subset(
            affine_subspace_product(a, b),
            affine_subspace_product(c, d)) =
        (affine_subspace_subset(a, c) and affine_subspace_subset(b, d)))
} by {
    if a.contains(p) and b.contains(q) {
        let product_subset = affine_subspace_subset(
            affine_subspace_product(a, b),
            affine_subspace_product(c, d))
        let coordinate_subset =
            affine_subspace_subset(a, c) and affine_subspace_subset(b, d)
        if product_subset {
            affine_subspace_subset_left_of_product_subset(
                a, b, c, d, q)
            affine_subspace_subset(a, c)
            affine_subspace_subset_right_of_product_subset(
                a, b, c, d, p)
            affine_subspace_subset(b, d)
            coordinate_subset
        }
        if coordinate_subset {
            affine_subspace_product_subset_of_subset_left_right(
                a, b, c, d)
            product_subset
        }
        product_subset = coordinate_subset
        affine_subspace_subset(
            affine_subspace_product(a, b),
            affine_subspace_product(c, d)) =
            (affine_subspace_subset(a, c) and
                affine_subspace_subset(b, d))
    }
}

/// Products of inhabited affine subspaces over matching ambient spaces are
/// equal exactly when their corresponding factors are equal.
theorem affine_subspace_product_eq_iff[V: AddCommGroup, W: AddCommGroup, P, Q](
    a: AffineSubspace[V, P],
    b: AffineSubspace[W, Q],
    c: AffineSubspace[V, P],
    d: AffineSubspace[W, Q],
    p: P,
    q: Q
) {
    a.space = c.space and b.space = d.space
        and a.contains(p) and b.contains(q) implies
        ((affine_subspace_product(a, b) =
            affine_subspace_product(c, d)) =
        (a = c and b = d))
} by {
    if a.space = c.space and b.space = d.space
        and a.contains(p) and b.contains(q) {
        if affine_subspace_product(a, b) =
            affine_subspace_product(c, d) {
            forall(x: Pair[P, Q]) {
                if affine_subspace_product(a, b).contains(x) {
                    affine_subspace_product(c, d).contains(x)
                }
            }
            affine_subspace_subset(
                affine_subspace_product(a, b),
                affine_subspace_product(c, d))
            affine_subspace_product_subset_iff(a, b, c, d, p, q)
            affine_subspace_subset(a, c)
            affine_subspace_subset(b, d)

            let pair = Pair.new(p, q)
            pair_new_first(p, q)
            pair_new_second(p, q)
            affine_subspace_product_contains_pair_iff(a, b, p, q)
            affine_subspace_product(a, b).contains(pair)
            affine_subspace_product(c, d).contains(pair)
            affine_subspace_product_contains_pair_iff(c, d, p, q)
            c.contains(p)
            d.contains(q)

            forall(x: Pair[P, Q]) {
                if affine_subspace_product(c, d).contains(x) {
                    affine_subspace_product(a, b).contains(x)
                }
            }
            affine_subspace_subset(
                affine_subspace_product(c, d),
                affine_subspace_product(a, b))
            affine_subspace_product_subset_iff(c, d, a, b, p, q)
            affine_subspace_subset(c, a)
            affine_subspace_subset(d, b)

            affine_subspace_subset_antisymm(a, c)
            a = c
            affine_subspace_subset_antisymm(b, d)
            b = d
            a = c and b = d
        }
        if a = c and b = d {
            a = c
            b = d
            affine_subspace_product(a, b) =
                affine_subspace_product(c, d)
        }
        (affine_subspace_product(a, b) =
            affine_subspace_product(c, d)) =
            (a = c and b = d)
    }
}

/// Products of nonempty affine subspaces over matching ambient spaces are
/// equal exactly when their corresponding factors are equal.
theorem affine_subspace_product_eq_iff_of_nonempty[V: AddCommGroup, W: AddCommGroup, P, Q](
    a: AffineSubspace[V, P],
    b: AffineSubspace[W, Q],
    c: AffineSubspace[V, P],
    d: AffineSubspace[W, Q]
) {
    a.space = c.space and b.space = d.space
        and affine_subspace_nonempty(a)
        and affine_subspace_nonempty(b) implies
        ((affine_subspace_product(a, b) =
            affine_subspace_product(c, d)) =
        (a = c and b = d))
} by {
    if a.space = c.space and b.space = d.space
        and affine_subspace_nonempty(a)
        and affine_subspace_nonempty(b) {
        affine_subspace_nonempty(a) = exists(x: P) {
            a.contains(x)
        }
        let p: P satisfy {
            a.contains(p)
        }
        affine_subspace_nonempty(b) = exists(y: Q) {
            b.contains(y)
        }
        let q: Q satisfy {
            b.contains(q)
        }
        affine_subspace_product_eq_iff(a, b, c, d, p, q)
        (affine_subspace_product(a, b) =
            affine_subspace_product(c, d)) =
            (a = c and b = d)
    }
}

/// A product with an empty first factor is the empty affine subspace of the
/// product affine space.
theorem affine_subspace_product_empty_left[V: AddCommGroup, W: AddCommGroup, P, Q](
    a: AffineSpace[V, P],
    b: AffineSubspace[W, Q]
) {
    affine_subspace_product(affine_subspace_empty(a), b) =
        affine_subspace_empty(affine_space_product(a, b.space))
} by {
    let empty_a = affine_subspace_empty(a)
    let product_space = affine_space_product(a, b.space)
    let lhs = affine_subspace_product(empty_a, b)
    let rhs = affine_subspace_empty(product_space)

    AffineSubspace[V, P].new(
        a, affine_subspace_empty_contains[P]) = Option.some(empty_a)
    empty_a.space = a
    affine_subspace_product_space(empty_a, b)
    lhs.space = product_space

    AffineSubspace[Pair[V, W], Pair[P, Q]].new(
        product_space, affine_subspace_empty_contains[Pair[P, Q]]) =
        Option.some(rhs)
    rhs.space = product_space

    forall(x: Pair[P, Q]) {
        affine_subspace_product_contains_iff(empty_a, b, x)
        affine_subspace_empty_not_contains(a, x.first)
        not lhs.contains(x)
        affine_subspace_empty_not_contains(product_space, x)
        not rhs.contains(x)
        lhs.contains(x) = rhs.contains(x)
    }
    affine_subspace_ext(lhs, rhs)
    lhs = rhs
}

/// A product with an empty second factor is the empty affine subspace of the
/// product affine space.
theorem affine_subspace_product_empty_right[V: AddCommGroup, W: AddCommGroup, P, Q](
    a: AffineSubspace[V, P],
    b: AffineSpace[W, Q]
) {
    affine_subspace_product(a, affine_subspace_empty(b)) =
        affine_subspace_empty(affine_space_product(a.space, b))
} by {
    let empty_b = affine_subspace_empty(b)
    let product_space = affine_space_product(a.space, b)
    let lhs = affine_subspace_product(a, empty_b)
    let rhs = affine_subspace_empty(product_space)

    AffineSubspace[W, Q].new(
        b, affine_subspace_empty_contains[Q]) = Option.some(empty_b)
    empty_b.space = b
    affine_subspace_product_space(a, empty_b)
    lhs.space = product_space

    AffineSubspace[Pair[V, W], Pair[P, Q]].new(
        product_space, affine_subspace_empty_contains[Pair[P, Q]]) =
        Option.some(rhs)
    rhs.space = product_space

    forall(x: Pair[P, Q]) {
        affine_subspace_product_contains_iff(a, empty_b, x)
        affine_subspace_empty_not_contains(b, x.second)
        not lhs.contains(x)
        affine_subspace_empty_not_contains(product_space, x)
        not rhs.contains(x)
        lhs.contains(x) = rhs.contains(x)
    }
    affine_subspace_ext(lhs, rhs)
    lhs = rhs
}

/// A product affine subspace is empty exactly when at least one factor is
/// empty.
theorem affine_subspace_product_eq_empty_iff[V: AddCommGroup, W: AddCommGroup, P, Q](
    a: AffineSubspace[V, P],
    b: AffineSubspace[W, Q]
) {
    (affine_subspace_product(a, b) =
        affine_subspace_empty(affine_space_product(a.space, b.space))) =
    ((a = affine_subspace_empty(a.space)) or
        (b = affine_subspace_empty(b.space)))
} by {
    let product = affine_subspace_product(a, b)
    let product_space = affine_space_product(a.space, b.space)
    let product_empty =
        product = affine_subspace_empty(product_space)
    let a_empty = a = affine_subspace_empty(a.space)
    let b_empty = b = affine_subspace_empty(b.space)
    let product_nonempty = affine_subspace_nonempty(product)
    let a_nonempty = affine_subspace_nonempty(a)
    let b_nonempty = affine_subspace_nonempty(b)

    affine_subspace_product_space(a, b)
    product.space = product_space
    affine_subspace_eq_empty_iff_not_nonempty(product, product_space)
    product_empty = not product_nonempty

    affine_subspace_product_nonempty_iff(a, b)
    product_nonempty = (a_nonempty and b_nonempty)
    not_and(a_nonempty, b_nonempty)
    not product_nonempty = (not a_nonempty or not b_nonempty)

    affine_subspace_eq_empty_iff_not_nonempty(a, a.space)
    a_empty = not a_nonempty
    affine_subspace_eq_empty_iff_not_nonempty(b, b.space)
    b_empty = not b_nonempty

    (not a_nonempty or not b_nonempty) = (a_empty or b_empty)
    if product_empty {
        not product_nonempty
        not a_nonempty or not b_nonempty
    }
    if not product_empty {
        product_nonempty
        a_nonempty and b_nonempty
        not (not a_nonempty or not b_nonempty)
    }
    product_empty = (not a_nonempty or not b_nonempty)
    product_empty = (a_empty or b_empty)
    (affine_subspace_product(a, b) =
        affine_subspace_empty(product_space)) = product_empty
    ((a = affine_subspace_empty(a.space)) or
        (b = affine_subspace_empty(b.space))) =
        (a_empty or b_empty)
    (affine_subspace_product(a, b) =
        affine_subspace_empty(product_space)) =
        ((a = affine_subspace_empty(a.space)) or
            (b = affine_subspace_empty(b.space)))
}

/// The product of two full affine subspaces is the full affine subspace of
/// the product affine space.
theorem affine_subspace_product_univ[V: AddCommGroup, W: AddCommGroup, P, Q](
    a: AffineSpace[V, P],
    b: AffineSpace[W, Q]
) {
    affine_subspace_product(
        affine_subspace_univ(a),
        affine_subspace_univ(b)) =
        affine_subspace_univ(affine_space_product(a, b))
} by {
    let univ_a = affine_subspace_univ(a)
    let univ_b = affine_subspace_univ(b)
    let product_space = affine_space_product(a, b)
    let lhs = affine_subspace_product(univ_a, univ_b)
    let rhs = affine_subspace_univ(product_space)

    AffineSubspace[V, P].new(
        a, affine_subspace_univ_contains[P]) = Option.some(univ_a)
    univ_a.space = a
    AffineSubspace[W, Q].new(
        b, affine_subspace_univ_contains[Q]) = Option.some(univ_b)
    univ_b.space = b
    affine_subspace_product_space(univ_a, univ_b)
    lhs.space = product_space

    AffineSubspace[Pair[V, W], Pair[P, Q]].new(
        product_space, affine_subspace_univ_contains[Pair[P, Q]]) =
        Option.some(rhs)
    rhs.space = product_space

    forall(x: Pair[P, Q]) {
        affine_subspace_univ_contains_all(a, x.first)
        affine_subspace_univ_contains_all(b, x.second)
        affine_subspace_product_contains_iff(univ_a, univ_b, x)
        lhs.contains(x)
        affine_subspace_univ_contains_all(product_space, x)
        rhs.contains(x)
        lhs.contains(x) = rhs.contains(x)
    }
    affine_subspace_ext(lhs, rhs)
    lhs = rhs
}

/// The empty affine subspace is contained in every affine subspace over the same affine space.
theorem affine_subspace_empty_subset[V: AddCommGroup, P](
    a: AffineSpace[V, P], b: AffineSubspace[V, P]) {
    affine_subspace_subset(affine_subspace_empty(a), b)
} by {
    forall(x: P) {
        if affine_subspace_empty(a).contains(x) {
            affine_subspace_empty_not_contains(a, x)
            false
        }
    }
}

/// Every affine subspace is contained in the full affine subspace over the same affine space.
theorem affine_subspace_subset_univ[V: AddCommGroup, P](
    a: AffineSpace[V, P], b: AffineSubspace[V, P]) {
    affine_subspace_subset(b, affine_subspace_univ(a))
} by {
    forall(x: P) {
        if b.contains(x) {
            affine_subspace_univ_contains_all(a, x)
            affine_subspace_univ(a).contains(x)
        }
    }
}

/// The membership predicate of the intersection of two affine subspaces, taken
/// to be empty when the two ambient spaces disagree.
define affine_subspace_intersection_total_contains[V: AddCommGroup, P](
    a: AffineSubspace[V, P], b: AffineSubspace[V, P], x: P) -> Bool {
    a.space = b.space and a.contains(x) and b.contains(x)
}

/// The intersection-contains predicate is closed under the affine combination
/// of the first ambient space.
theorem affine_subspace_intersection_total_constraint[V: AddCommGroup, P](
    a: AffineSubspace[V, P], b: AffineSubspace[V, P]) {
    affine_subspace_constraint(a.space,
        affine_subspace_intersection_total_contains(a, b))
} by {
    forall(p: P, q: P, r: P) {
        if affine_subspace_intersection_total_contains(a, b, p)
           and affine_subspace_intersection_total_contains(a, b, q)
           and affine_subspace_intersection_total_contains(a, b, r) {
            affine_subspace_intersection_total_contains(a, b, p) =
                (a.space = b.space and a.contains(p) and b.contains(p))
            affine_subspace_intersection_total_contains(a, b, q) =
                (a.space = b.space and a.contains(q) and b.contains(q))
            affine_subspace_intersection_total_contains(a, b, r) =
                (a.space = b.space and a.contains(r) and b.contains(r))
            affine_subspace_closed(a, p, q, r)
            affine_subspace_closed(b, p, q, r)
            let s: P = a.space.vadd(a.space.vsub(p, q), r)
            b.contains(s)
            affine_subspace_intersection_total_contains(a, b,
                a.space.vadd(a.space.vsub(p, q), r))
        }
    }
}

/// The intersection of two affine subspaces; empty when the ambient spaces differ.
let affine_subspace_intersection[V: AddCommGroup, P](a: AffineSubspace[V, P], b: AffineSubspace[V, P]) -> result: AffineSubspace[V, P] satisfy {
    AffineSubspace[V, P].new(a.space, affine_subspace_intersection_total_contains(a, b)) =
        Option.some(result)
} by {
    affine_subspace_intersection_total_constraint(a, b)
}

/// The intersection's ambient space is the ambient space of the first.
theorem affine_subspace_intersection_space[V: AddCommGroup, P](
    a: AffineSubspace[V, P], b: AffineSubspace[V, P]) {
    affine_subspace_intersection(a, b).space = a.space
}

/// The intersection's membership predicate is the conjunction of the two predicates,
/// when the ambient spaces agree.
theorem affine_subspace_intersection_contains_iff[V: AddCommGroup, P](
    a: AffineSubspace[V, P], b: AffineSubspace[V, P], x: P) {
    a.space = b.space implies
        (affine_subspace_intersection(a, b).contains(x) =
            (a.contains(x) and b.contains(x)))
} by {
    affine_subspace_intersection(a, b).contains =
        affine_subspace_intersection_total_contains(a, b)
    if a.space = b.space {
        affine_subspace_intersection_total_contains(a, b, x) =
            (a.contains(x) and b.contains(x))
    }
}

/// The intersection of two affine subspaces is contained in the first.
theorem affine_subspace_intersection_subset_left[V: AddCommGroup, P](
    a: AffineSubspace[V, P], b: AffineSubspace[V, P]) {
    a.space = b.space implies
        affine_subspace_subset(affine_subspace_intersection(a, b), a)
} by {
    if a.space = b.space {
        forall(x: P) {
            if affine_subspace_intersection(a, b).contains(x) {
                affine_subspace_intersection_contains_iff(a, b, x)
                a.contains(x)
            }
        }
    }
}

/// The intersection of two affine subspaces is contained in the second.
theorem affine_subspace_intersection_subset_right[V: AddCommGroup, P](
    a: AffineSubspace[V, P], b: AffineSubspace[V, P]) {
    a.space = b.space implies
        affine_subspace_subset(affine_subspace_intersection(a, b), b)
} by {
    if a.space = b.space {
        forall(x: P) {
            if affine_subspace_intersection(a, b).contains(x) {
                affine_subspace_intersection_contains_iff(a, b, x)
                b.contains(x)
            }
        }
    }
}

/// Any affine subspace contained in two affine subspaces is contained in their intersection.
theorem affine_subspace_subset_intersection[V: AddCommGroup, P](
    a: AffineSubspace[V, P], b: AffineSubspace[V, P], c: AffineSubspace[V, P]) {
    a.space = b.space and a.space = c.space and
        affine_subspace_subset(a, b) and affine_subspace_subset(a, c)
        implies affine_subspace_subset(a, affine_subspace_intersection(b, c))
} by {
    if a.space = b.space and a.space = c.space and
        affine_subspace_subset(a, b) and affine_subspace_subset(a, c) {
        affine_subspace_subset(a, b) = forall(x: P) {
            a.contains(x) implies b.contains(x)
        }
        affine_subspace_subset(a, c) = forall(x: P) {
            a.contains(x) implies c.contains(x)
        }
        forall(x: P) {
            if a.contains(x) {
                b.contains(x)
                c.contains(x)
                affine_subspace_intersection_contains_iff(b, c, x)
                affine_subspace_intersection(b, c).contains(x)
            }
        }
    }
}

/// Products commute with intersections of affine subspaces over matching
/// ambient spaces.
theorem affine_subspace_product_intersection[V: AddCommGroup, W: AddCommGroup, P, Q](
    a: AffineSubspace[V, P],
    b: AffineSubspace[W, Q],
    c: AffineSubspace[V, P],
    d: AffineSubspace[W, Q]
) {
    a.space = c.space and b.space = d.space implies
        affine_subspace_product(
            affine_subspace_intersection(a, c),
            affine_subspace_intersection(b, d)) =
        affine_subspace_intersection(
            affine_subspace_product(a, b),
            affine_subspace_product(c, d))
} by {
    if a.space = c.space and b.space = d.space {
        let ac = affine_subspace_intersection(a, c)
        let bd = affine_subspace_intersection(b, d)
        let ab = affine_subspace_product(a, b)
        let cd = affine_subspace_product(c, d)
        let lhs = affine_subspace_product(ac, bd)
        let rhs = affine_subspace_intersection(ab, cd)

        affine_subspace_intersection_space(a, c)
        ac.space = a.space
        affine_subspace_intersection_space(b, d)
        bd.space = b.space
        affine_subspace_product_space(ac, bd)
        lhs.space = affine_space_product(a.space, b.space)

        affine_subspace_product_space(a, b)
        ab.space = affine_space_product(a.space, b.space)
        affine_subspace_product_space(c, d)
        cd.space = affine_space_product(c.space, d.space)
        ab.space = cd.space
        affine_subspace_intersection_space(ab, cd)
        rhs.space = ab.space
        lhs.space = rhs.space

        forall(x: Pair[P, Q]) {
            let coordinate_intersection =
                (a.contains(x.first) and c.contains(x.first)) and
                (b.contains(x.second) and d.contains(x.second))
            let product_intersection =
                (a.contains(x.first) and b.contains(x.second)) and
                (c.contains(x.first) and d.contains(x.second))

            affine_subspace_intersection_contains_iff(a, c, x.first)
            ac.contains(x.first) =
                (a.contains(x.first) and c.contains(x.first))
            affine_subspace_intersection_contains_iff(b, d, x.second)
            bd.contains(x.second) =
                (b.contains(x.second) and d.contains(x.second))
            affine_subspace_product_contains_iff(ac, bd, x)
            lhs.contains(x) =
                (ac.contains(x.first) and bd.contains(x.second))
            lhs.contains(x) = coordinate_intersection

            affine_subspace_product_contains_iff(a, b, x)
            ab.contains(x) =
                (a.contains(x.first) and b.contains(x.second))
            affine_subspace_product_contains_iff(c, d, x)
            cd.contains(x) =
                (c.contains(x.first) and d.contains(x.second))
            affine_subspace_intersection_contains_iff(ab, cd, x)
            rhs.contains(x) = (ab.contains(x) and cd.contains(x))
            rhs.contains(x) = product_intersection

            coordinate_intersection = product_intersection
            lhs.contains(x) = rhs.contains(x)
        }
        affine_subspace_ext(lhs, rhs)
        lhs = rhs
        affine_subspace_product(ac, bd) = lhs
        affine_subspace_intersection(ab, cd) = rhs
        affine_subspace_product(ac, bd) =
            affine_subspace_intersection(ab, cd)
        affine_subspace_product(
            affine_subspace_intersection(a, c),
            affine_subspace_intersection(b, d)) =
            affine_subspace_intersection(
                affine_subspace_product(a, b),
                affine_subspace_product(c, d))
    }
}

/// The membership predicate of the affine subspace consisting of just the
/// point `p`.
define affine_subspace_singleton_contains[P](p: P, x: P) -> Bool {
    x = p
}

/// The singleton predicate satisfies the affine-subspace closure property.
theorem affine_subspace_singleton_constraint[V: AddCommGroup, P](
    a: AffineSpace[V, P], p: P) {
    affine_subspace_constraint(a, affine_subspace_singleton_contains[P](p))
} by {
    forall(x: P, y: P, z: P) {
        if affine_subspace_singleton_contains[P](p, x)
           and affine_subspace_singleton_contains[P](p, y)
           and affine_subspace_singleton_contains[P](p, z) {
            x = p
            y = p
            z = p
            affine_vsub_self(a, p)
            affine_zero_vadd(a, p)
            affine_subspace_singleton_contains[P](p, a.vadd(a.vsub(x, y), z))
        }
    }
}

/// The affine subspace consisting of the single point `p`.
let affine_subspace_singleton[V: AddCommGroup, P](a: AffineSpace[V, P], p: P) -> result: AffineSubspace[V, P] satisfy {
    AffineSubspace[V, P].new(a, affine_subspace_singleton_contains[P](p)) =
        Option.some(result)
} by {
    affine_subspace_singleton_constraint(a, p)
}

/// The singleton affine subspace's ambient space is the given affine space.
theorem affine_subspace_singleton_space[V: AddCommGroup, P](
    a: AffineSpace[V, P], p: P) {
    affine_subspace_singleton(a, p).space = a
}

/// A point belongs to the singleton subspace exactly when it equals the point.
theorem affine_subspace_singleton_contains_iff[V: AddCommGroup, P](
    a: AffineSpace[V, P], p: P, x: P) {
    affine_subspace_singleton(a, p).contains(x) = (x = p)
} by {
    affine_subspace_singleton(a, p).contains = affine_subspace_singleton_contains[P](p)
}

/// The singleton subspace contains its defining point.
theorem affine_subspace_singleton_contains_self[V: AddCommGroup, P](
    a: AffineSpace[V, P], p: P) {
    affine_subspace_singleton(a, p).contains(p)
} by {
    affine_subspace_singleton_contains_iff(a, p, p)
}

/// The product of two singleton affine subspaces is the singleton affine
/// subspace at the corresponding pair.
theorem affine_subspace_product_singleton[V: AddCommGroup, W: AddCommGroup, P, Q](
    a: AffineSpace[V, P],
    b: AffineSpace[W, Q],
    p: P,
    q: Q
) {
    affine_subspace_product(
        affine_subspace_singleton(a, p),
        affine_subspace_singleton(b, q)) =
        affine_subspace_singleton(
            affine_space_product(a, b), Pair.new(p, q))
} by {
    let singleton_p = affine_subspace_singleton(a, p)
    let singleton_q = affine_subspace_singleton(b, q)
    let pair = Pair.new(p, q)
    let product_space = affine_space_product(a, b)
    let lhs = affine_subspace_product(singleton_p, singleton_q)
    let rhs = affine_subspace_singleton(product_space, pair)

    affine_subspace_singleton_space(a, p)
    singleton_p.space = a
    affine_subspace_singleton_space(b, q)
    singleton_q.space = b
    affine_subspace_product_space(singleton_p, singleton_q)
    lhs.space = product_space
    affine_subspace_singleton_space(product_space, pair)
    rhs.space = product_space

    pair_new_first(p, q)
    pair_new_second(p, q)
    forall(x: Pair[P, Q]) {
        let coordinate_eq = x.first = p and x.second = q
        if coordinate_eq {
            x.first = pair.first
            x.second = pair.second
            pair_ext(x, pair)
            x = pair
        }
        if x = pair {
            x.first = p
            x.second = q
            coordinate_eq
        }
        coordinate_eq = (x = pair)

        affine_subspace_singleton_contains_iff(a, p, x.first)
        singleton_p.contains(x.first) = (x.first = p)
        affine_subspace_singleton_contains_iff(b, q, x.second)
        singleton_q.contains(x.second) = (x.second = q)
        affine_subspace_product_contains_iff(singleton_p, singleton_q, x)
        lhs.contains(x) = coordinate_eq

        affine_subspace_singleton_contains_iff(product_space, pair, x)
        rhs.contains(x) = (x = pair)
        lhs.contains(x) = rhs.contains(x)
    }
    affine_subspace_ext(lhs, rhs)
    lhs = rhs
    affine_subspace_product(singleton_p, singleton_q) =
        affine_subspace_singleton(product_space, pair)
    affine_subspace_product(
        affine_subspace_singleton(a, p),
        affine_subspace_singleton(b, q)) =
        affine_subspace_singleton(
            affine_space_product(a, b), Pair.new(p, q))
}

/// The membership predicate of the affine span of a set: a point is in the
/// span exactly when it belongs to every affine subspace over `a` that
/// contains the source set.
define affine_span_contains[V: AddCommGroup, P](
    a: AffineSpace[V, P], src: P -> Bool, x: P) -> Bool {
    forall(s: AffineSubspace[V, P]) {
        s.space = a and (forall(y: P) { src(y) implies s.contains(y) })
            implies s.contains(x)
    }
}

/// The affine-span membership predicate satisfies the affine-subspace closure property.
theorem affine_span_constraint[V: AddCommGroup, P](
    a: AffineSpace[V, P], src: P -> Bool) {
    affine_subspace_constraint(a, affine_span_contains(a, src))
} by {
    forall(p: P, q: P, r: P) {
        if affine_span_contains(a, src, p)
           and affine_span_contains(a, src, q)
           and affine_span_contains(a, src, r) {
            affine_span_contains(a, src, p) = forall(s: AffineSubspace[V, P]) {
                s.space = a and (forall(y: P) { src(y) implies s.contains(y) })
                    implies s.contains(p)
            }
            affine_span_contains(a, src, q) = forall(s: AffineSubspace[V, P]) {
                s.space = a and (forall(y: P) { src(y) implies s.contains(y) })
                    implies s.contains(q)
            }
            affine_span_contains(a, src, r) = forall(s: AffineSubspace[V, P]) {
                s.space = a and (forall(y: P) { src(y) implies s.contains(y) })
                    implies s.contains(r)
            }
            forall(s: AffineSubspace[V, P]) {
                if s.space = a and (forall(y: P) { src(y) implies s.contains(y) }) {
                    s.contains(q)
                    s.contains(r)
                    affine_subspace_closed(s, p, q, r)
                    s.contains(s.space.vadd(s.space.vsub(p, q), r))
                    s.contains(a.vadd(a.vsub(p, q), r))
                }
            }
            affine_span_contains(a, src, a.vadd(a.vsub(p, q), r))
        }
    }
}

/// The affine span of a set in an affine space: the smallest affine subspace
/// containing the set.
let affine_span[V: AddCommGroup, P](a: AffineSpace[V, P], src: P -> Bool) -> result: AffineSubspace[V, P] satisfy {
    AffineSubspace[V, P].new(a, affine_span_contains(a, src)) =
        Option.some(result)
} by {
    affine_span_constraint(a, src)
}

/// The affine span's ambient space is the given affine space.
theorem affine_span_space[V: AddCommGroup, P](
    a: AffineSpace[V, P], src: P -> Bool) {
    affine_span(a, src).space = a
}

/// Membership in the affine span unfolds to the universal predicate.
theorem affine_span_contains_iff[V: AddCommGroup, P](
    a: AffineSpace[V, P], src: P -> Bool, x: P) {
    affine_span(a, src).contains(x) = affine_span_contains(a, src, x)
} by {
    affine_span(a, src).contains = affine_span_contains(a, src)
}

/// Every member of the source set lies in its affine span.
theorem affine_span_contains_src[V: AddCommGroup, P](
    a: AffineSpace[V, P], src: P -> Bool, x: P) {
    src(x) implies affine_span(a, src).contains(x)
} by {
    if src(x) {
        affine_span_contains_iff(a, src, x)
        forall(s: AffineSubspace[V, P]) {
            if s.space = a and (forall(y: P) { src(y) implies s.contains(y) }) {
                s.contains(x)
            }
        }
        affine_span_contains(a, src, x)
    }
}

/// The affine span is the smallest affine subspace containing the source set:
/// any affine subspace over the same ambient space that contains the source
/// set also contains the span.
theorem affine_span_subset[V: AddCommGroup, P](
    a: AffineSpace[V, P], src: P -> Bool, t: AffineSubspace[V, P]) {
    t.space = a and (forall(y: P) { src(y) implies t.contains(y) })
        implies affine_subspace_subset(affine_span(a, src), t)
} by {
    if t.space = a and (forall(y: P) { src(y) implies t.contains(y) }) {
        forall(x: P) {
            if affine_span(a, src).contains(x) {
                affine_span_contains_iff(a, src, x)
                affine_span_contains(a, src, x) = forall(s: AffineSubspace[V, P]) {
                    s.space = a and (forall(y: P) { src(y) implies s.contains(y) })
                        implies s.contains(x)
                }
                t.contains(x)
            }
        }
    }
}

/// An affine span is contained in an affine subspace exactly when the source
/// set is contained in that subspace, provided the ambient spaces agree.
theorem affine_span_subset_iff[V: AddCommGroup, P](
    a: AffineSpace[V, P], src: P -> Bool, t: AffineSubspace[V, P]
) {
    t.space = a implies
        (affine_subspace_subset(affine_span(a, src), t) =
            forall(y: P) { src(y) implies t.contains(y) })
} by {
    if t.space = a {
        let span_subset = affine_subspace_subset(affine_span(a, src), t)
        let src_subset = forall(y: P) { src(y) implies t.contains(y) }
        if span_subset {
            affine_subspace_subset(affine_span(a, src), t) = forall(x: P) {
                affine_span(a, src).contains(x) implies t.contains(x)
            }
            forall(y: P) {
                if src(y) {
                    affine_span_contains_src(a, src, y)
                    t.contains(y)
                }
            }
            src_subset
        }
        if src_subset {
            forall(y: P) {
                src(y) implies t.contains(y)
            }
            affine_span_subset(a, src, t)
            span_subset
        }
        span_subset = src_subset
        affine_subspace_subset(affine_span(a, src), t) =
            forall(y: P) { src(y) implies t.contains(y) }
    }
}

/// The affine span of the membership predicate of an affine subspace is that subspace.
theorem affine_span_subspace_eq[V: AddCommGroup, P](s: AffineSubspace[V, P]) {
    affine_span(s.space, s.contains) = s
} by {
    let span = affine_span(s.space, s.contains)
    affine_span_space(s.space, s.contains)
    span.space = s.space
    forall(y: P) {
        s.contains(y) implies s.contains(y)
    }
    affine_span_subset(s.space, s.contains, s)
    affine_subspace_subset(span, s)
    affine_subspace_subset(span, s) = forall(x: P) {
        span.contains(x) implies s.contains(x)
    }
    forall(x: P) {
        if span.contains(x) {
            s.contains(x)
        }
        if s.contains(x) {
            affine_span_contains_src(s.space, s.contains, x)
            span.contains(x)
        }
        span.contains(x) = s.contains(x)
    }
    affine_subspace_ext(span, s)
}

/// Taking the affine span of the membership predicate of an affine span does
/// not change that affine span.
theorem affine_span_idempotent[V: AddCommGroup, P](
    a: AffineSpace[V, P], src: P -> Bool
) {
    affine_span(a, affine_span(a, src).contains) = affine_span(a, src)
} by {
    let span = affine_span(a, src)
    affine_span_space(a, src)
    span.space = a
    affine_span_subspace_eq(span)
}

/// A predicate is the membership predicate of its affine span exactly when it
/// is closed under affine combinations.
theorem affine_span_fixed_iff[V: AddCommGroup, P](
    a: AffineSpace[V, P], src: P -> Bool
) {
    (affine_span(a, src).contains = src) =
        affine_subspace_constraint(a, src)
} by {
    let span = affine_span(a, src)
    if span.contains = src {
        affine_span_constraint(a, src)
    }
    if affine_subspace_constraint(a, src) {
        let s: AffineSubspace[V, P] satisfy {
            AffineSubspace[V, P].new(a, src) = Option.some(s)
        }
        s.space = a
        s.contains = src
        affine_span_subspace_eq(s)
        affine_span(a, src) = s
        span.contains = src
    }
}

/// The intersection of two source predicates for affine spans.
define affine_span_intersection_src[P](
    src1: P -> Bool, src2: P -> Bool, x: P
) -> Bool {
    src1(x) and src2(x)
}

/// Membership in the intersection source predicate is membership in both
/// sources.
theorem affine_span_intersection_src_iff[P](
    src1: P -> Bool, src2: P -> Bool, x: P
) {
    affine_span_intersection_src(src1, src2, x) =
        (src1(x) and src2(x))
}

/// Intersection source predicates are commutative.
theorem affine_span_intersection_src_comm[P](
    src1: P -> Bool, src2: P -> Bool, x: P
) {
    affine_span_intersection_src(src1, src2, x) =
        affine_span_intersection_src(src2, src1, x)
} by {
    affine_span_intersection_src_iff(src1, src2, x)
    affine_span_intersection_src_iff(src2, src1, x)
    and_comm(src1(x), src2(x))
}

/// Intersection source predicates are associative.
theorem affine_span_intersection_src_assoc[P](
    src1: P -> Bool, src2: P -> Bool, src3: P -> Bool, x: P
) {
    affine_span_intersection_src(
        affine_span_intersection_src(src1, src2), src3, x) =
        affine_span_intersection_src(
            src1, affine_span_intersection_src(src2, src3), x)
} by {
    affine_span_intersection_src_iff(
        affine_span_intersection_src(src1, src2), src3, x)
    affine_span_intersection_src_iff(src1, src2, x)
    affine_span_intersection_src_iff(
        src1, affine_span_intersection_src(src2, src3), x)
    affine_span_intersection_src_iff(src2, src3, x)
    and_assoc(src1(x), src2(x), src3(x))
}

/// Intersecting a source predicate with itself does not change it.
theorem affine_span_intersection_src_idempotent[P](
    src: P -> Bool, x: P
) {
    affine_span_intersection_src(src, src, x) = src(x)
} by {
    affine_span_intersection_src_iff(src, src, x)
    and_self(src(x))
}

/// The intersection of two predicates closed under affine combinations is
/// closed under affine combinations.
theorem affine_span_intersection_src_constraint[V: AddCommGroup, P](
    a: AffineSpace[V, P],
    src1: P -> Bool,
    src2: P -> Bool
) {
    affine_subspace_constraint(a, src1)
        and affine_subspace_constraint(a, src2) implies
        affine_subspace_constraint(
            a, affine_span_intersection_src(src1, src2))
} by {
    if affine_subspace_constraint(a, src1)
        and affine_subspace_constraint(a, src2) {
        affine_subspace_constraint(a, src1) = forall(p: P, q: P, r: P) {
            src1(p) and src1(q) and src1(r) implies
                src1(a.vadd(a.vsub(p, q), r))
        }
        affine_subspace_constraint(a, src2) = forall(p: P, q: P, r: P) {
            src2(p) and src2(q) and src2(r) implies
                src2(a.vadd(a.vsub(p, q), r))
        }
        forall(p: P, q: P, r: P) {
            if affine_span_intersection_src(src1, src2, p)
                and affine_span_intersection_src(src1, src2, q)
                and affine_span_intersection_src(src1, src2, r) {
                affine_span_intersection_src_iff(src1, src2, p)
                affine_span_intersection_src_iff(src1, src2, q)
                affine_span_intersection_src_iff(src1, src2, r)
                src1(a.vadd(a.vsub(p, q), r))
                src2(a.vadd(a.vsub(p, q), r))
                affine_span_intersection_src_iff(
                    src1, src2, a.vadd(a.vsub(p, q), r))
                affine_span_intersection_src(
                    src1, src2, a.vadd(a.vsub(p, q), r))
            }
        }
    }
}

/// The intersection of two predicates fixed by affine span is fixed by affine
/// span.
theorem affine_span_intersection_fixed[V: AddCommGroup, P](
    a: AffineSpace[V, P],
    src1: P -> Bool,
    src2: P -> Bool
) {
    affine_span(a, src1).contains = src1
        and affine_span(a, src2).contains = src2 implies
        affine_span(a, affine_span_intersection_src(src1, src2)).contains =
            affine_span_intersection_src(src1, src2)
} by {
    if affine_span(a, src1).contains = src1
        and affine_span(a, src2).contains = src2 {
        affine_span_fixed_iff(a, src1)
        affine_subspace_constraint(a, src1)
        affine_span_fixed_iff(a, src2)
        affine_subspace_constraint(a, src2)
        affine_span_intersection_src_constraint(a, src1, src2)
        affine_subspace_constraint(
            a, affine_span_intersection_src(src1, src2))
        affine_span_fixed_iff(
            a, affine_span_intersection_src(src1, src2))
        affine_span(a, affine_span_intersection_src(src1, src2)).contains =
            affine_span_intersection_src(src1, src2)
    }
}

/// The union of two source predicates for affine spans.
define affine_span_union_src[P](
    src1: P -> Bool, src2: P -> Bool, x: P
) -> Bool {
    src1(x) or src2(x)
}

/// Membership in the union source predicate is membership in either source.
theorem affine_span_union_src_iff[P](
    src1: P -> Bool, src2: P -> Bool, x: P
) {
    affine_span_union_src(src1, src2, x) = (src1(x) or src2(x))
}

/// Unioning a source predicate with itself does not change it.
theorem affine_span_union_src_idempotent[P](
    src: P -> Bool, x: P
) {
    affine_span_union_src(src, src, x) = src(x)
} by {
    affine_span_union_src_iff(src, src, x)
    or_self(src(x))
}

/// A source predicate absorbs its union with another source under
/// intersection.
theorem affine_span_intersection_union_absorption[P](
    src1: P -> Bool, src2: P -> Bool, x: P
) {
    affine_span_intersection_src(
        src1, affine_span_union_src(src1, src2), x) = src1(x)
} by {
    affine_span_intersection_src_iff(
        src1, affine_span_union_src(src1, src2), x)
    affine_span_union_src_iff(src1, src2, x)
}

/// A source predicate absorbs its intersection with another source under
/// union.
theorem affine_span_union_intersection_absorption[P](
    src1: P -> Bool, src2: P -> Bool, x: P
) {
    affine_span_union_src(
        src1, affine_span_intersection_src(src1, src2), x) = src1(x)
} by {
    affine_span_union_src_iff(
        src1, affine_span_intersection_src(src1, src2), x)
    affine_span_intersection_src_iff(src1, src2, x)
}

/// Intersection distributes over union for source predicates.
theorem affine_span_intersection_union_distrib[P](
    src1: P -> Bool, src2: P -> Bool, src3: P -> Bool, x: P
) {
    affine_span_intersection_src(
        src1, affine_span_union_src(src2, src3), x) =
        affine_span_union_src(
            affine_span_intersection_src(src1, src2),
            affine_span_intersection_src(src1, src3), x)
} by {
    affine_span_intersection_src_iff(
        src1, affine_span_union_src(src2, src3), x)
    affine_span_union_src_iff(src2, src3, x)
    affine_span_union_src_iff(
        affine_span_intersection_src(src1, src2),
        affine_span_intersection_src(src1, src3), x)
    affine_span_intersection_src_iff(src1, src2, x)
    affine_span_intersection_src_iff(src1, src3, x)
    and_or_distrib_left(src1(x), src2(x), src3(x))
}

/// Union distributes over intersection for source predicates.
theorem affine_span_union_intersection_distrib[P](
    src1: P -> Bool, src2: P -> Bool, src3: P -> Bool, x: P
) {
    affine_span_union_src(
        src1, affine_span_intersection_src(src2, src3), x) =
        affine_span_intersection_src(
            affine_span_union_src(src1, src2),
            affine_span_union_src(src1, src3), x)
} by {
    affine_span_union_src_iff(
        src1, affine_span_intersection_src(src2, src3), x)
    affine_span_intersection_src_iff(src2, src3, x)
    affine_span_intersection_src_iff(
        affine_span_union_src(src1, src2),
        affine_span_union_src(src1, src3), x)
    affine_span_union_src_iff(src1, src2, x)
    affine_span_union_src_iff(src1, src3, x)
    or_and_distrib_left(src1(x), src2(x), src3(x))
}

/// Monotonicity of the affine span: enlarging the source set enlarges the span.
theorem affine_span_mono[V: AddCommGroup, P](
    a: AffineSpace[V, P], src1: P -> Bool, src2: P -> Bool) {
    (forall(y: P) { src1(y) implies src2(y) })
        implies affine_subspace_subset(affine_span(a, src1), affine_span(a, src2))
} by {
    if forall(y: P) { src1(y) implies src2(y) } {
        forall(y: P) {
            if src1(y) {
                affine_span_contains_src(a, src2, y)
                affine_span(a, src2).contains(y)
            }
        }
        affine_span_space(a, src2)
        affine_span_subset(a, src1, affine_span(a, src2))
    }
}

/// The affine span of an intersection source is contained in the affine span
/// of its left source.
theorem affine_span_intersection_subset_left[V: AddCommGroup, P](
    a: AffineSpace[V, P], src1: P -> Bool, src2: P -> Bool
) {
    affine_subspace_subset(
        affine_span(a, affine_span_intersection_src(src1, src2)),
        affine_span(a, src1))
} by {
    forall(x: P) {
        if affine_span_intersection_src(src1, src2, x) {
            affine_span_intersection_src_iff(src1, src2, x)
            src1(x)
        }
    }
    affine_span_mono(a, affine_span_intersection_src(src1, src2), src1)
}

/// The affine span of an intersection source is contained in the affine span
/// of its right source.
theorem affine_span_intersection_subset_right[V: AddCommGroup, P](
    a: AffineSpace[V, P], src1: P -> Bool, src2: P -> Bool
) {
    affine_subspace_subset(
        affine_span(a, affine_span_intersection_src(src1, src2)),
        affine_span(a, src2))
} by {
    forall(x: P) {
        if affine_span_intersection_src(src1, src2, x) {
            affine_span_intersection_src_iff(src1, src2, x)
            src2(x)
        }
    }
    affine_span_mono(a, affine_span_intersection_src(src1, src2), src2)
}

/// Enlarging both sources enlarges their intersection source.
theorem affine_span_intersection_src_mono[P](
    src1: P -> Bool,
    src2: P -> Bool,
    dst1: P -> Bool,
    dst2: P -> Bool
) {
    ((forall(x: P) { src1(x) implies dst1(x) })
    and (forall(x: P) { src2(x) implies dst2(x) })) implies
    forall(x: P) {
        affine_span_intersection_src(src1, src2, x) implies
            affine_span_intersection_src(dst1, dst2, x)
    }
} by {
    if (forall(x: P) { src1(x) implies dst1(x) })
    and (forall(x: P) { src2(x) implies dst2(x) }) {
        forall(x: P) {
            if affine_span_intersection_src(src1, src2, x) {
                affine_span_intersection_src_iff(src1, src2, x)
                src1(x)
                src2(x)
                dst1(x)
                dst2(x)
                affine_span_intersection_src_iff(dst1, dst2, x)
                affine_span_intersection_src(dst1, dst2, x)
            }
        }
    }
}

/// Enlarging both sources of an intersection enlarges its affine span.
theorem affine_span_intersection_mono[V: AddCommGroup, P](
    a: AffineSpace[V, P],
    src1: P -> Bool,
    src2: P -> Bool,
    dst1: P -> Bool,
    dst2: P -> Bool
) {
    ((forall(x: P) { src1(x) implies dst1(x) })
    and (forall(x: P) { src2(x) implies dst2(x) })) implies
    affine_subspace_subset(
        affine_span(a, affine_span_intersection_src(src1, src2)),
        affine_span(a, affine_span_intersection_src(dst1, dst2)))
} by {
    if (forall(x: P) { src1(x) implies dst1(x) })
    and (forall(x: P) { src2(x) implies dst2(x) }) {
        affine_span_intersection_src_mono(src1, src2, dst1, dst2)
        affine_span_mono(
            a, affine_span_intersection_src(src1, src2),
            affine_span_intersection_src(dst1, dst2))
    }
}

/// Intersection-source affine spans are equal when each pair of component
/// sources contains the other.
theorem affine_span_intersection_eq_of_mutual_src_containment[V: AddCommGroup, P](
    a: AffineSpace[V, P],
    src1: P -> Bool,
    src2: P -> Bool,
    dst1: P -> Bool,
    dst2: P -> Bool
) {
    (((forall(x: P) { src1(x) implies dst1(x) })
        and (forall(x: P) { src2(x) implies dst2(x) }))
    and ((forall(x: P) { dst1(x) implies src1(x) })
        and (forall(x: P) { dst2(x) implies src2(x) }))) implies
    affine_span(a, affine_span_intersection_src(src1, src2)) =
        affine_span(a, affine_span_intersection_src(dst1, dst2))
} by {
    if ((forall(x: P) { src1(x) implies dst1(x) })
        and (forall(x: P) { src2(x) implies dst2(x) }))
    and ((forall(x: P) { dst1(x) implies src1(x) })
        and (forall(x: P) { dst2(x) implies src2(x) })) {
        let src_span =
            affine_span(a, affine_span_intersection_src(src1, src2))
        let dst_span =
            affine_span(a, affine_span_intersection_src(dst1, dst2))

        affine_span_intersection_mono(a, src1, src2, dst1, dst2)
        affine_subspace_subset(src_span, dst_span)
        affine_span_intersection_mono(a, dst1, dst2, src1, src2)
        affine_subspace_subset(dst_span, src_span)

        affine_span_space(a, affine_span_intersection_src(src1, src2))
        src_span.space = a
        affine_span_space(a, affine_span_intersection_src(dst1, dst2))
        dst_span.space = a
        src_span.space = dst_span.space
        (src_span.space = dst_span.space
            and affine_subspace_subset(src_span, dst_span)
            and affine_subspace_subset(dst_span, src_span))
        affine_subspace_subset_antisymm(src_span, dst_span)
        src_span = dst_span
        src_span =
            affine_span(a, affine_span_intersection_src(src1, src2))
        dst_span =
            affine_span(a, affine_span_intersection_src(dst1, dst2))
        affine_span(a, affine_span_intersection_src(src1, src2)) =
            affine_span(a, affine_span_intersection_src(dst1, dst2))
    }
}

/// If the left-source span is contained in the intersection-source span,
/// then the two spans are equal.
theorem affine_span_intersection_eq_left_of_subset[V: AddCommGroup, P](
    a: AffineSpace[V, P], src1: P -> Bool, src2: P -> Bool
) {
    affine_subspace_subset(
        affine_span(a, src1),
        affine_span(a, affine_span_intersection_src(src1, src2))) implies
    affine_span(a, affine_span_intersection_src(src1, src2)) =
        affine_span(a, src1)
} by {
    let intersection_span =
        affine_span(a, affine_span_intersection_src(src1, src2))
    let left_span = affine_span(a, src1)

    if affine_subspace_subset(left_span, intersection_span) {
        affine_span_space(
            a, affine_span_intersection_src(src1, src2))
        intersection_span.space = a
        affine_span_space(a, src1)
        left_span.space = a
        intersection_span.space = left_span.space
        affine_span_intersection_subset_left(a, src1, src2)
        affine_subspace_subset(intersection_span, left_span)
        (intersection_span.space = left_span.space
            and affine_subspace_subset(intersection_span, left_span)
            and affine_subspace_subset(left_span, intersection_span))
        affine_subspace_subset_antisymm(intersection_span, left_span)
        intersection_span = left_span
        intersection_span =
            affine_span(a, affine_span_intersection_src(src1, src2))
        left_span = affine_span(a, src1)
        affine_span(a, affine_span_intersection_src(src1, src2)) =
            affine_span(a, src1)
    }
}

/// Equality with the left-source span implies containment of the left span
/// in the intersection-source span.
theorem affine_span_left_subset_intersection_of_eq[V: AddCommGroup, P](
    a: AffineSpace[V, P], src1: P -> Bool, src2: P -> Bool
) {
    affine_span(a, affine_span_intersection_src(src1, src2)) =
        affine_span(a, src1) implies
    affine_subspace_subset(
        affine_span(a, src1),
        affine_span(a, affine_span_intersection_src(src1, src2)))
} by {
    let intersection_span =
        affine_span(a, affine_span_intersection_src(src1, src2))
    let left_span = affine_span(a, src1)
    if intersection_span = left_span {
        affine_subspace_subset_refl(left_span)
        affine_subspace_subset(left_span, intersection_span)
    }
}

/// An intersection-source span equals the span of its left source exactly
/// when the left span is contained in the intersection-source span.
theorem affine_span_intersection_eq_left_iff[V: AddCommGroup, P](
    a: AffineSpace[V, P], src1: P -> Bool, src2: P -> Bool
) {
    affine_subspace_subset(
        affine_span(a, src1),
        affine_span(a, affine_span_intersection_src(src1, src2))) =
    (affine_span(a, affine_span_intersection_src(src1, src2)) =
        affine_span(a, src1))
} by {
    if affine_subspace_subset(
        affine_span(a, src1),
        affine_span(a, affine_span_intersection_src(src1, src2))) {
        affine_span_intersection_eq_left_of_subset(a, src1, src2)
    }
    if affine_span(a, affine_span_intersection_src(src1, src2)) =
        affine_span(a, src1) {
        affine_span_left_subset_intersection_of_eq(a, src1, src2)
    }
}

/// If the right-source span is contained in the intersection-source span,
/// then the two spans are equal.
theorem affine_span_intersection_eq_right_of_subset[V: AddCommGroup, P](
    a: AffineSpace[V, P], src1: P -> Bool, src2: P -> Bool
) {
    affine_subspace_subset(
        affine_span(a, src2),
        affine_span(a, affine_span_intersection_src(src1, src2))) implies
    affine_span(a, affine_span_intersection_src(src1, src2)) =
        affine_span(a, src2)
} by {
    let intersection_span =
        affine_span(a, affine_span_intersection_src(src1, src2))
    let right_span = affine_span(a, src2)

    if affine_subspace_subset(right_span, intersection_span) {
        affine_span_space(
            a, affine_span_intersection_src(src1, src2))
        intersection_span.space = a
        affine_span_space(a, src2)
        right_span.space = a
        intersection_span.space = right_span.space
        affine_span_intersection_subset_right(a, src1, src2)
        affine_subspace_subset(intersection_span, right_span)
        (intersection_span.space = right_span.space
            and affine_subspace_subset(intersection_span, right_span)
            and affine_subspace_subset(right_span, intersection_span))
        affine_subspace_subset_antisymm(intersection_span, right_span)
        intersection_span = right_span
        intersection_span =
            affine_span(a, affine_span_intersection_src(src1, src2))
        right_span = affine_span(a, src2)
        affine_span(a, affine_span_intersection_src(src1, src2)) =
            affine_span(a, src2)
    }
}

/// Equality with the right-source span implies containment of the right span
/// in the intersection-source span.
theorem affine_span_right_subset_intersection_of_eq[V: AddCommGroup, P](
    a: AffineSpace[V, P], src1: P -> Bool, src2: P -> Bool
) {
    affine_span(a, affine_span_intersection_src(src1, src2)) =
        affine_span(a, src2) implies
    affine_subspace_subset(
        affine_span(a, src2),
        affine_span(a, affine_span_intersection_src(src1, src2)))
} by {
    let intersection_span =
        affine_span(a, affine_span_intersection_src(src1, src2))
    let right_span = affine_span(a, src2)
    if intersection_span = right_span {
        affine_subspace_subset_refl(right_span)
        affine_subspace_subset(right_span, intersection_span)
    }
}

/// An intersection-source span equals the span of its right source exactly
/// when the right span is contained in the intersection-source span.
theorem affine_span_intersection_eq_right_iff[V: AddCommGroup, P](
    a: AffineSpace[V, P], src1: P -> Bool, src2: P -> Bool
) {
    affine_subspace_subset(
        affine_span(a, src2),
        affine_span(a, affine_span_intersection_src(src1, src2))) =
    (affine_span(a, affine_span_intersection_src(src1, src2)) =
        affine_span(a, src2))
} by {
    if affine_subspace_subset(
        affine_span(a, src2),
        affine_span(a, affine_span_intersection_src(src1, src2))) {
        affine_span_intersection_eq_right_of_subset(a, src1, src2)
    }
    if affine_span(a, affine_span_intersection_src(src1, src2)) =
        affine_span(a, src2) {
        affine_span_right_subset_intersection_of_eq(a, src1, src2)
    }
}

/// The affine span of the left source is contained in the affine span of the
/// union.
theorem affine_span_subset_union_left[V: AddCommGroup, P](
    a: AffineSpace[V, P], src1: P -> Bool, src2: P -> Bool
) {
    affine_subspace_subset(
        affine_span(a, src1),
        affine_span(a, affine_span_union_src(src1, src2)))
} by {
    forall(x: P) {
        if src1(x) {
            affine_span_union_src_iff(src1, src2, x)
            affine_span_union_src(src1, src2, x)
        }
    }
    affine_span_mono(a, src1, affine_span_union_src(src1, src2))
}

/// The affine span of the right source is contained in the affine span of the
/// union.
theorem affine_span_subset_union_right[V: AddCommGroup, P](
    a: AffineSpace[V, P], src1: P -> Bool, src2: P -> Bool
) {
    affine_subspace_subset(
        affine_span(a, src2),
        affine_span(a, affine_span_union_src(src1, src2)))
} by {
    forall(x: P) {
        if src2(x) {
            affine_span_union_src_iff(src1, src2, x)
            affine_span_union_src(src1, src2, x)
        }
    }
    affine_span_mono(a, src2, affine_span_union_src(src1, src2))
}

/// The affine span of a union is contained in an affine subspace exactly when
/// both component spans are contained in that subspace.
theorem affine_span_union_subset_iff[V: AddCommGroup, P](
    a: AffineSpace[V, P],
    src1: P -> Bool,
    src2: P -> Bool,
    t: AffineSubspace[V, P]
) {
    t.space = a implies
        (affine_subspace_subset(
            affine_span(a, affine_span_union_src(src1, src2)), t) =
        (affine_subspace_subset(affine_span(a, src1), t)
            and affine_subspace_subset(affine_span(a, src2), t)))
} by {
    if t.space = a {
        let union_span =
            affine_span(a, affine_span_union_src(src1, src2))
        let left_span = affine_span(a, src1)
        let right_span = affine_span(a, src2)
        let union_subset = affine_subspace_subset(union_span, t)
        let both_subset = (affine_subspace_subset(left_span, t)
            and affine_subspace_subset(right_span, t))
        if union_subset {
            affine_span_subset_union_left(a, src1, src2)
            affine_subspace_subset(left_span, union_span)
            affine_subspace_subset_trans(left_span, union_span, t)
            affine_subspace_subset(left_span, t)
            affine_span_subset_union_right(a, src1, src2)
            affine_subspace_subset(right_span, union_span)
            affine_subspace_subset_trans(right_span, union_span, t)
            affine_subspace_subset(right_span, t)
            both_subset
        }
        if both_subset {
            affine_subspace_subset(left_span, t)
            affine_subspace_subset(right_span, t)
            affine_subspace_subset(left_span, t) = forall(x: P) {
                left_span.contains(x) implies t.contains(x)
            }
            affine_subspace_subset(right_span, t) = forall(x: P) {
                right_span.contains(x) implies t.contains(x)
            }
            forall(x: P) {
                if affine_span_union_src(src1, src2, x) {
                    affine_span_union_src_iff(src1, src2, x)
                    if src1(x) {
                        affine_span_contains_src(a, src1, x)
                        left_span.contains(x)
                        t.contains(x)
                    } else {
                        src2(x)
                        affine_span_contains_src(a, src2, x)
                        right_span.contains(x)
                        t.contains(x)
                    }
                }
            }
            affine_span_subset(a, affine_span_union_src(src1, src2), t)
            union_subset
        }
        union_subset = both_subset
        affine_subspace_subset(
            affine_span(a, affine_span_union_src(src1, src2)), t) =
        (affine_subspace_subset(affine_span(a, src1), t)
            and affine_subspace_subset(affine_span(a, src2), t))
    }
}

/// Pointwise equivalent source predicates have the same affine span.
theorem affine_span_congr[V: AddCommGroup, P](
    a: AffineSpace[V, P], src1: P -> Bool, src2: P -> Bool
) {
    (forall(y: P) { src1(y) = src2(y) }) implies
        affine_span(a, src1) = affine_span(a, src2)
} by {
    if forall(y: P) { src1(y) = src2(y) } {
        forall(y: P) {
            src1(y) implies src2(y)
        }
        affine_span_mono(a, src1, src2)
        let left = affine_span(a, src1)
        let right = affine_span(a, src2)
        affine_subspace_subset(left, right)

        forall(y: P) {
            src2(y) implies src1(y)
        }
        affine_span_mono(a, src2, src1)
        affine_subspace_subset(right, left)

        affine_subspace_subset(left, right) = forall(x: P) {
            left.contains(x) implies right.contains(x)
        }
        affine_subspace_subset(right, left) = forall(x: P) {
            right.contains(x) implies left.contains(x)
        }
        forall(x: P) {
            left.contains(x) = right.contains(x)
        }
        affine_span_space(a, src1)
        affine_span_space(a, src2)
        left.space = right.space
        affine_subspace_ext(left, right)
        left = right
    }
}

/// The affine span of a union is unchanged when the two source predicates are
/// exchanged.
theorem affine_span_union_comm[V: AddCommGroup, P](
    a: AffineSpace[V, P], src1: P -> Bool, src2: P -> Bool
) {
    affine_span(a, affine_span_union_src(src1, src2)) =
        affine_span(a, affine_span_union_src(src2, src1))
} by {
    forall(x: P) {
        affine_span_union_src_iff(src1, src2, x)
        affine_span_union_src_iff(src2, src1, x)
        or_comm(src1(x), src2(x))
        affine_span_union_src(src1, src2, x) =
            affine_span_union_src(src2, src1, x)
    }
    affine_span_congr(
        a, affine_span_union_src(src1, src2),
        affine_span_union_src(src2, src1))
}

/// The affine span of three unioned source predicates is independent of the
/// placement of parentheses.
theorem affine_span_union_assoc[V: AddCommGroup, P](
    a: AffineSpace[V, P],
    src1: P -> Bool,
    src2: P -> Bool,
    src3: P -> Bool
) {
    affine_span(a,
        affine_span_union_src(affine_span_union_src(src1, src2), src3)) =
    affine_span(a,
        affine_span_union_src(src1, affine_span_union_src(src2, src3)))
} by {
    let left_src =
        affine_span_union_src(affine_span_union_src(src1, src2), src3)
    let right_src =
        affine_span_union_src(src1, affine_span_union_src(src2, src3))
    forall(x: P) {
        affine_span_union_src_iff(
            affine_span_union_src(src1, src2), src3, x)
        affine_span_union_src_iff(src1, src2, x)
        affine_span_union_src_iff(
            src1, affine_span_union_src(src2, src3), x)
        affine_span_union_src_iff(src2, src3, x)
        or_assoc(src1(x), src2(x), src3(x))
        left_src(x) = right_src(x)
    }
    affine_span_congr(a, left_src, right_src)
    affine_span(a, left_src) = affine_span(a, right_src)
}

/// The affine span of an intersection is unchanged when the two source
/// predicates are exchanged.
theorem affine_span_intersection_comm[V: AddCommGroup, P](
    a: AffineSpace[V, P], src1: P -> Bool, src2: P -> Bool
) {
    affine_span(a, affine_span_intersection_src(src1, src2)) =
        affine_span(a, affine_span_intersection_src(src2, src1))
} by {
    forall(x: P) {
        affine_span_intersection_src_comm(src1, src2, x)
    }
    affine_span_congr(
        a, affine_span_intersection_src(src1, src2),
        affine_span_intersection_src(src2, src1))
}

/// The affine span of three intersected source predicates is independent of
/// the placement of parentheses.
theorem affine_span_intersection_assoc[V: AddCommGroup, P](
    a: AffineSpace[V, P],
    src1: P -> Bool,
    src2: P -> Bool,
    src3: P -> Bool
) {
    affine_span(a,
        affine_span_intersection_src(
            affine_span_intersection_src(src1, src2), src3)) =
    affine_span(a,
        affine_span_intersection_src(
            src1, affine_span_intersection_src(src2, src3)))
} by {
    let left_src =
        affine_span_intersection_src(
            affine_span_intersection_src(src1, src2), src3)
    let right_src =
        affine_span_intersection_src(
            src1, affine_span_intersection_src(src2, src3))
    forall(x: P) {
        affine_span_intersection_src_assoc(src1, src2, src3, x)
        left_src(x) = right_src(x)
    }
    affine_span_congr(a, left_src, right_src)
    affine_span(a, left_src) = affine_span(a, right_src)
}

/// Intersecting a source predicate with itself does not change its affine
/// span.
theorem affine_span_intersection_idempotent[V: AddCommGroup, P](
    a: AffineSpace[V, P], src: P -> Bool
) {
    affine_span(a, affine_span_intersection_src(src, src)) =
        affine_span(a, src)
} by {
    forall(x: P) {
        affine_span_intersection_src_idempotent(src, x)
    }
    affine_span_congr(a, affine_span_intersection_src(src, src), src)
}

/// Unioning a source predicate with itself does not change its affine span.
theorem affine_span_union_idempotent[V: AddCommGroup, P](
    a: AffineSpace[V, P], src: P -> Bool
) {
    affine_span(a, affine_span_union_src(src, src)) =
        affine_span(a, src)
} by {
    forall(x: P) {
        affine_span_union_src_idempotent(src, x)
    }
    affine_span_congr(a, affine_span_union_src(src, src), src)
}

/// Intersecting a source predicate with its union with another source does
/// not change its affine span.
theorem affine_span_intersection_union_absorption_eq[V: AddCommGroup, P](
    a: AffineSpace[V, P], src1: P -> Bool, src2: P -> Bool
) {
    affine_span(a, affine_span_intersection_src(
        src1, affine_span_union_src(src1, src2))) =
        affine_span(a, src1)
} by {
    forall(x: P) {
        affine_span_intersection_union_absorption(src1, src2, x)
    }
    affine_span_congr(
        a,
        affine_span_intersection_src(
            src1, affine_span_union_src(src1, src2)),
        src1)
}

/// Unioning a source predicate with its intersection with another source does
/// not change its affine span.
theorem affine_span_union_intersection_absorption_eq[V: AddCommGroup, P](
    a: AffineSpace[V, P], src1: P -> Bool, src2: P -> Bool
) {
    affine_span(a, affine_span_union_src(
        src1, affine_span_intersection_src(src1, src2))) =
        affine_span(a, src1)
} by {
    forall(x: P) {
        affine_span_union_intersection_absorption(src1, src2, x)
    }
    affine_span_congr(
        a,
        affine_span_union_src(
            src1, affine_span_intersection_src(src1, src2)),
        src1)
}

/// Distributing intersection over union in source predicates does not change
/// their affine span.
theorem affine_span_intersection_union_distrib_eq[V: AddCommGroup, P](
    a: AffineSpace[V, P],
    src1: P -> Bool,
    src2: P -> Bool,
    src3: P -> Bool
) {
    affine_span(a, affine_span_intersection_src(
        src1, affine_span_union_src(src2, src3))) =
    affine_span(a, affine_span_union_src(
        affine_span_intersection_src(src1, src2),
        affine_span_intersection_src(src1, src3)))
} by {
    let left_src = affine_span_intersection_src(
        src1, affine_span_union_src(src2, src3))
    let right_src = affine_span_union_src(
        affine_span_intersection_src(src1, src2),
        affine_span_intersection_src(src1, src3))
    forall(x: P) {
        affine_span_intersection_union_distrib(src1, src2, src3, x)
        left_src(x) = right_src(x)
    }
    affine_span_congr(a, left_src, right_src)
    affine_span(a, left_src) = affine_span(a, right_src)
}

/// Distributing union over intersection in source predicates does not change
/// their affine span.
theorem affine_span_union_intersection_distrib_eq[V: AddCommGroup, P](
    a: AffineSpace[V, P],
    src1: P -> Bool,
    src2: P -> Bool,
    src3: P -> Bool
) {
    affine_span(a, affine_span_union_src(
        src1, affine_span_intersection_src(src2, src3))) =
    affine_span(a, affine_span_intersection_src(
        affine_span_union_src(src1, src2),
        affine_span_union_src(src1, src3)))
} by {
    let left_src = affine_span_union_src(
        src1, affine_span_intersection_src(src2, src3))
    let right_src = affine_span_intersection_src(
        affine_span_union_src(src1, src2),
        affine_span_union_src(src1, src3))
    forall(x: P) {
        affine_span_union_intersection_distrib(src1, src2, src3, x)
        left_src(x) = right_src(x)
    }
    affine_span_congr(a, left_src, right_src)
    affine_span(a, left_src) = affine_span(a, right_src)
}

/// Two affine spans are equal exactly when each source set is contained in
/// the affine span of the other source set.
theorem affine_span_eq_iff[V: AddCommGroup, P](
    a: AffineSpace[V, P], src1: P -> Bool, src2: P -> Bool
) {
    (affine_span(a, src1) = affine_span(a, src2)) =
        ((forall(y: P) { src1(y) implies affine_span(a, src2).contains(y) })
        and (forall(y: P) { src2(y) implies affine_span(a, src1).contains(y) }))
} by {
    let left = affine_span(a, src1)
    let right = affine_span(a, src2)
    let forward = forall(y: P) { src1(y) implies right.contains(y) }
    let backward = forall(y: P) { src2(y) implies left.contains(y) }
    let mutual = forward and backward
    if left = right {
        forall(y: P) {
            if src1(y) {
                affine_span_contains_src(a, src1, y)
                right.contains(y)
            }
        }
        forward
        forall(y: P) {
            if src2(y) {
                affine_span_contains_src(a, src2, y)
                left.contains(y)
            }
        }
        backward
        mutual
    }
    if mutual {
        forward
        backward
        affine_span_space(a, src1)
        affine_span_space(a, src2)
        right.space = a
        right.space = a and forward
        affine_span_subset(a, src1, right)
        left.space = a
        left.space = a and backward
        affine_span_subset(a, src2, left)
        affine_subspace_subset(left, right)
        affine_subspace_subset(right, left)
        affine_subspace_subset(left, right) = forall(x: P) {
            left.contains(x) implies right.contains(x)
        }
        affine_subspace_subset(right, left) = forall(x: P) {
            right.contains(x) implies left.contains(x)
        }
        forall(x: P) {
            if left.contains(x) {
                right.contains(x)
            }
            if right.contains(x) {
                left.contains(x)
            }
            left.contains(x) = right.contains(x)
        }
        left.space = right.space
        affine_subspace_ext(left, right)
        left = right
    }
    (left = right) = mutual
    (affine_span(a, src1) = affine_span(a, src2)) =
        ((forall(y: P) { src1(y) implies affine_span(a, src2).contains(y) })
        and (forall(y: P) { src2(y) implies affine_span(a, src1).contains(y) }))
}

/// Replacing the left source of a union by its affine-span membership
/// predicate does not change the affine span of the union.
theorem affine_span_union_left_closure[V: AddCommGroup, P](
    a: AffineSpace[V, P], src1: P -> Bool, src2: P -> Bool
) {
    affine_span(a, affine_span_union_src(affine_span(a, src1).contains, src2)) =
        affine_span(a, affine_span_union_src(src1, src2))
} by {
    let left_span = affine_span(a, src1)
    let expanded_src = affine_span_union_src(left_span.contains, src2)
    let original_src = affine_span_union_src(src1, src2)
    let expanded = affine_span(a, expanded_src)
    let original = affine_span(a, original_src)
    let forward = forall(x: P) {
        expanded_src(x) implies original.contains(x)
    }
    let backward = forall(x: P) {
        original_src(x) implies expanded.contains(x)
    }
    forall(x: P) {
        if expanded_src(x) {
            affine_span_union_src_iff(left_span.contains, src2, x)
            if left_span.contains(x) {
                affine_span_subset_union_left(a, src1, src2)
                affine_subspace_subset(left_span, original)
                affine_subspace_subset(left_span, original) = forall(y: P) {
                    left_span.contains(y) implies original.contains(y)
                }
                original.contains(x)
            } else {
                src2(x)
                original_src(x)
                affine_span_contains_src(a, original_src, x)
                original.contains(x)
            }
        }
    }
    forward
    forall(x: P) {
        if original_src(x) {
            affine_span_union_src_iff(src1, src2, x)
            if src1(x) {
                affine_span_contains_src(a, src1, x)
                left_span.contains(x)
                expanded_src(x)
                affine_span_contains_src(a, expanded_src, x)
                expanded.contains(x)
            } else {
                src2(x)
                expanded_src(x)
                affine_span_contains_src(a, expanded_src, x)
                expanded.contains(x)
            }
        }
    }
    backward
    forward and backward
    affine_span_eq_iff(a, expanded_src, original_src)
    (affine_span(a, expanded_src) = affine_span(a, original_src)) =
        (forward and backward)
    expanded = original
    affine_span(a,
        affine_span_union_src(affine_span(a, src1).contains, src2)) =
        affine_span(a, affine_span_union_src(src1, src2))
}

/// Replacing the right source of a union by its affine-span membership
/// predicate does not change the affine span of the union.
theorem affine_span_union_right_closure[V: AddCommGroup, P](
    a: AffineSpace[V, P], src1: P -> Bool, src2: P -> Bool
) {
    affine_span(a, affine_span_union_src(src1, affine_span(a, src2).contains)) =
        affine_span(a, affine_span_union_src(src1, src2))
} by {
    let right_span = affine_span(a, src2)
    let expanded_src = affine_span_union_src(src1, right_span.contains)
    let original_src = affine_span_union_src(src1, src2)
    let expanded = affine_span(a, expanded_src)
    let original = affine_span(a, original_src)
    let forward = forall(x: P) {
        expanded_src(x) implies original.contains(x)
    }
    let backward = forall(x: P) {
        original_src(x) implies expanded.contains(x)
    }
    forall(x: P) {
        if expanded_src(x) {
            affine_span_union_src_iff(src1, right_span.contains, x)
            if src1(x) {
                original_src(x)
                affine_span_contains_src(a, original_src, x)
                original.contains(x)
            } else {
                right_span.contains(x)
                affine_span_subset_union_right(a, src1, src2)
                affine_subspace_subset(right_span, original)
                affine_subspace_subset(right_span, original) = forall(y: P) {
                    right_span.contains(y) implies original.contains(y)
                }
                original.contains(x)
            }
        }
    }
    forward
    forall(x: P) {
        if original_src(x) {
            affine_span_union_src_iff(src1, src2, x)
            if src1(x) {
                expanded_src(x)
                affine_span_contains_src(a, expanded_src, x)
                expanded.contains(x)
            } else {
                src2(x)
                affine_span_contains_src(a, src2, x)
                right_span.contains(x)
                expanded_src(x)
                affine_span_contains_src(a, expanded_src, x)
                expanded.contains(x)
            }
        }
    }
    backward
    forward and backward
    affine_span_eq_iff(a, expanded_src, original_src)
    (affine_span(a, expanded_src) = affine_span(a, original_src)) =
        (forward and backward)
    expanded = original
    affine_span(a,
        affine_span_union_src(src1, affine_span(a, src2).contains)) =
        affine_span(a, affine_span_union_src(src1, src2))
}

/// The least affine subspace containing two affine subspaces over the left
/// subspace's ambient affine space.
define affine_subspace_sup[V: AddCommGroup, P](
    a: AffineSubspace[V, P], b: AffineSubspace[V, P]
) -> AffineSubspace[V, P] {
    affine_span(a.space, affine_span_union_src(a.contains, b.contains))
}

/// The step adjoining an affine subspace to a finite join.
define affine_subspace_list_sup_step[V: AddCommGroup, P](
    item: AffineSubspace[V, P],
    acc: AffineSubspace[V, P]
) -> AffineSubspace[V, P] {
    affine_subspace_sup(item, acc)
}

/// The join of a distinguished affine subspace and a finite list of further
/// affine subspaces.
define affine_subspace_list_sup_nonempty[V: AddCommGroup, P](
    head: AffineSubspace[V, P],
    tail: List[AffineSubspace[V, P]]
) -> AffineSubspace[V, P] {
    fold_right[AffineSubspace[V, P], AffineSubspace[V, P]](
        tail, affine_subspace_list_sup_step[V, P], head)
}

/// The nonempty finite join with no further affine subspaces is its
/// distinguished affine subspace.
theorem affine_subspace_list_sup_nonempty_nil[V: AddCommGroup, P](
    head: AffineSubspace[V, P]
) {
    affine_subspace_list_sup_nonempty(
        head, List.nil[AffineSubspace[V, P]]) = head
} by {
}

/// Adjoining an affine subspace at the front of the tail joins it with the
/// remaining nonempty finite join.
theorem affine_subspace_list_sup_nonempty_cons[V: AddCommGroup, P](
    head: AffineSubspace[V, P],
    item: AffineSubspace[V, P],
    tail: List[AffineSubspace[V, P]]
) {
    affine_subspace_list_sup_nonempty(head, List.cons(item, tail)) =
        affine_subspace_sup(
            item, affine_subspace_list_sup_nonempty(head, tail))
} by {
}

/// The join of two affine subspaces is the affine span of the union of their
/// membership predicates.
theorem affine_subspace_sup_eq_span_union[V: AddCommGroup, P](
    a: AffineSubspace[V, P], b: AffineSubspace[V, P]
) {
    affine_subspace_sup(a, b) =
        affine_span(a.space, affine_span_union_src(a.contains, b.contains))
}

/// The join of two affine spans is the affine span of the union of their
/// source predicates.
theorem affine_subspace_sup_affine_spans_eq_span_union[V: AddCommGroup, P](
    a: AffineSpace[V, P], src1: P -> Bool, src2: P -> Bool
) {
    affine_subspace_sup(affine_span(a, src1), affine_span(a, src2)) =
        affine_span(a, affine_span_union_src(src1, src2))
} by {
    let left_span = affine_span(a, src1)
    let right_span = affine_span(a, src2)
    affine_span_space(a, src1)
    left_span.space = a
    affine_subspace_sup_eq_span_union(left_span, right_span)
    affine_subspace_sup(left_span, right_span) =
        affine_span(
            a, affine_span_union_src(
                left_span.contains, right_span.contains))

    affine_span_union_left_closure(a, src1, right_span.contains)
    affine_span(
        a, affine_span_union_src(
            left_span.contains, right_span.contains)) =
        affine_span(
            a, affine_span_union_src(src1, right_span.contains))

    affine_span_union_right_closure(a, src1, src2)
    affine_span(
        a, affine_span_union_src(src1, right_span.contains)) =
        affine_span(a, affine_span_union_src(src1, src2))

    affine_subspace_sup(left_span, right_span) =
        affine_span(a, affine_span_union_src(src1, src2))
}

/// The ambient affine space of a join is the left subspace's ambient affine
/// space.
theorem affine_subspace_sup_space[V: AddCommGroup, P](
    a: AffineSubspace[V, P], b: AffineSubspace[V, P]
) {
    affine_subspace_sup(a, b).space = a.space
} by {
    affine_span_space(a.space, affine_span_union_src(a.contains, b.contains))
}

/// A nonempty finite join has the distinguished affine subspace's ambient
/// space when every further affine subspace has that same ambient space.
theorem affine_subspace_list_sup_nonempty_space[V: AddCommGroup, P](
    head: AffineSubspace[V, P],
    tail: List[AffineSubspace[V, P]]
) {
    (forall(item: AffineSubspace[V, P]) {
        tail.contains(item) implies item.space = head.space
    }) implies
    affine_subspace_list_sup_nonempty(head, tail).space = head.space
} by {
    define same_space(items: List[AffineSubspace[V, P]]) -> Bool {
        forall(item: AffineSubspace[V, P]) {
            items.contains(item) implies item.space = head.space
        }
    }
    define p(items: List[AffineSubspace[V, P]]) -> Bool {
        same_space(items) implies
        affine_subspace_list_sup_nonempty(head, items).space = head.space
    }

    affine_subspace_list_sup_nonempty_nil(head)
    affine_subspace_list_sup_nonempty(
        head, List.nil[AffineSubspace[V, P]]) = head
    same_space(List.nil[AffineSubspace[V, P]])
    p(List.nil[AffineSubspace[V, P]])

    forall(item: AffineSubspace[V, P],
        items: List[AffineSubspace[V, P]]) {
        if p(items) {
            if same_space(List.cons(item, items)) {
                same_space(List.cons(item, items)) =
                    forall(s: AffineSubspace[V, P]) {
                        List.cons(item, items).contains(s) implies
                            s.space = head.space
                    }
                List.cons(item, items).contains(item)
                item.space = head.space
                forall(s: AffineSubspace[V, P]) {
                    if items.contains(s) {
                        List.cons(item, items).contains(s)
                        s.space = head.space
                    }
                }
                same_space(items)
                p(items) = (same_space(items) implies
                    affine_subspace_list_sup_nonempty(
                        head, items).space = head.space)
                affine_subspace_list_sup_nonempty(head, items).space =
                    head.space
                item.space =
                    affine_subspace_list_sup_nonempty(head, items).space
                affine_subspace_sup_space(
                    item, affine_subspace_list_sup_nonempty(head, items))
                affine_subspace_sup(
                    item,
                    affine_subspace_list_sup_nonempty(head, items)).space =
                    item.space
                affine_subspace_list_sup_nonempty_cons(head, item, items)
                affine_subspace_list_sup_nonempty(
                    head, List.cons(item, items)).space = head.space
            }
            p(List.cons(item, items)) =
                (same_space(List.cons(item, items)) implies
                    affine_subspace_list_sup_nonempty(
                        head, List.cons(item, items)).space = head.space)
            p(List.cons(item, items))
        }
    }

    List.induction(function(items: List[AffineSubspace[V, P]]) {
        p(items)
    })
    p(tail)
    same_space(tail) = forall(item: AffineSubspace[V, P]) {
        tail.contains(item) implies item.space = head.space
    }
    p(tail) = (same_space(tail) implies
        affine_subspace_list_sup_nonempty(head, tail).space = head.space)
}

/// The left affine subspace is contained in the join.
theorem affine_subspace_subset_sup_left[V: AddCommGroup, P](
    a: AffineSubspace[V, P], b: AffineSubspace[V, P]
) {
    affine_subspace_subset(a, affine_subspace_sup(a, b))
} by {
    forall(x: P) {
        if a.contains(x) {
            affine_span_union_src_iff(a.contains, b.contains, x)
            affine_span_union_src(a.contains, b.contains, x)
            affine_span_contains_src(
                a.space, affine_span_union_src(a.contains, b.contains), x)
            affine_subspace_sup(a, b).contains(x)
        }
    }
}

/// The right affine subspace is contained in the join.
theorem affine_subspace_subset_sup_right[V: AddCommGroup, P](
    a: AffineSubspace[V, P], b: AffineSubspace[V, P]
) {
    affine_subspace_subset(b, affine_subspace_sup(a, b))
} by {
    forall(x: P) {
        if b.contains(x) {
            affine_span_union_src_iff(a.contains, b.contains, x)
            affine_span_union_src(a.contains, b.contains, x)
            affine_span_contains_src(
                a.space, affine_span_union_src(a.contains, b.contains), x)
            affine_subspace_sup(a, b).contains(x)
        }
    }
}

/// The distinguished affine subspace is contained in its nonempty finite
/// join.
theorem affine_subspace_subset_list_sup_nonempty_head[V: AddCommGroup, P](
    head: AffineSubspace[V, P],
    tail: List[AffineSubspace[V, P]]
) {
    affine_subspace_subset(
        head, affine_subspace_list_sup_nonempty(head, tail))
} by {
    define p(items: List[AffineSubspace[V, P]]) -> Bool {
        affine_subspace_subset(
            head, affine_subspace_list_sup_nonempty(head, items))
    }

    affine_subspace_list_sup_nonempty_nil(head)
    affine_subspace_subset_refl(head)
    p(List.nil[AffineSubspace[V, P]])

    forall(item: AffineSubspace[V, P],
        items: List[AffineSubspace[V, P]]) {
        if p(items) {
            let rest =
                affine_subspace_list_sup_nonempty(head, items)
            affine_subspace_subset(head, rest)
            affine_subspace_subset_sup_right(item, rest)
            affine_subspace_subset(
                rest, affine_subspace_sup(item, rest))
            affine_subspace_subset_trans(
                head, rest, affine_subspace_sup(item, rest))
            affine_subspace_subset(
                head, affine_subspace_sup(item, rest))
            affine_subspace_list_sup_nonempty_cons(head, item, items)
            p(List.cons(item, items))
        }
    }

    List.induction(function(items: List[AffineSubspace[V, P]]) {
        p(items)
    })
    p(tail)
}

/// Every affine subspace in the tail is contained in the corresponding
/// nonempty finite join.
theorem affine_subspace_subset_list_sup_nonempty_tail[V: AddCommGroup, P](
    head: AffineSubspace[V, P],
    tail: List[AffineSubspace[V, P]],
    member: AffineSubspace[V, P]
) {
    tail.contains(member) implies
        affine_subspace_subset(
            member, affine_subspace_list_sup_nonempty(head, tail))
} by {
    define p(items: List[AffineSubspace[V, P]]) -> Bool {
        forall(candidate: AffineSubspace[V, P]) {
            items.contains(candidate) implies
                affine_subspace_subset(
                    candidate,
                    affine_subspace_list_sup_nonempty(head, items))
        }
    }

    p(List.nil[AffineSubspace[V, P]])

    forall(item: AffineSubspace[V, P],
        items: List[AffineSubspace[V, P]]) {
        if p(items) {
            let rest =
                affine_subspace_list_sup_nonempty(head, items)
            let joined = affine_subspace_sup(item, rest)
            affine_subspace_list_sup_nonempty_cons(head, item, items)
            forall(s: AffineSubspace[V, P]) {
                if List.cons(item, items).contains(s) {
                    List.cons(item, items).contains(s) =
                        (item = s or items.contains(s))
                    if item = s {
                        affine_subspace_subset_sup_left(item, rest)
                        affine_subspace_subset(s, joined)
                    } else {
                        items.contains(s)
                        p(items) = forall(candidate: AffineSubspace[V, P]) {
                            items.contains(candidate) implies
                                affine_subspace_subset(candidate, rest)
                        }
                        affine_subspace_subset(s, rest)
                        affine_subspace_subset_sup_right(item, rest)
                        affine_subspace_subset(rest, joined)
                        affine_subspace_subset_trans(s, rest, joined)
                        affine_subspace_subset(s, joined)
                    }
                    affine_subspace_subset(
                        s,
                        affine_subspace_list_sup_nonempty(
                            head, List.cons(item, items)))
                }
            }
            p(List.cons(item, items))
        }
    }

    List.induction(function(items: List[AffineSubspace[V, P]]) {
        p(items)
    })
    p(tail)
    p(tail) = forall(s: AffineSubspace[V, P]) {
        tail.contains(s) implies
            affine_subspace_subset(
                s, affine_subspace_list_sup_nonempty(head, tail))
    }
}

/// The join is contained in every common upper bound over the same ambient
/// affine space.
theorem affine_subspace_sup_subset_of_subset_left_right[V: AddCommGroup, P](
    a: AffineSubspace[V, P],
    b: AffineSubspace[V, P],
    c: AffineSubspace[V, P]
) {
    c.space = a.space and affine_subspace_subset(a, c)
        and affine_subspace_subset(b, c)
        implies affine_subspace_subset(affine_subspace_sup(a, b), c)
} by {
    if c.space = a.space and affine_subspace_subset(a, c)
        and affine_subspace_subset(b, c) {
        affine_subspace_subset(a, c) = forall(x: P) {
            a.contains(x) implies c.contains(x)
        }
        affine_subspace_subset(b, c) = forall(x: P) {
            b.contains(x) implies c.contains(x)
        }
        forall(x: P) {
            if affine_span_union_src(a.contains, b.contains, x) {
                affine_span_union_src_iff(a.contains, b.contains, x)
                if a.contains(x) {
                    c.contains(x)
                } else {
                    b.contains(x)
                    c.contains(x)
                }
            }
        }
        affine_span_subset(
            a.space, affine_span_union_src(a.contains, b.contains), c)
        affine_subspace_subset(
            affine_span(
                a.space, affine_span_union_src(a.contains, b.contains)),
            c)
        affine_subspace_sup_eq_span_union(a, b)
        affine_subspace_subset(affine_subspace_sup(a, b), c)
    }
}

/// A join is contained in an affine subspace over the same ambient space
/// exactly when both component subspaces are contained in it.
theorem affine_subspace_sup_subset_iff[V: AddCommGroup, P](
    a: AffineSubspace[V, P],
    b: AffineSubspace[V, P],
    c: AffineSubspace[V, P]
) {
    c.space = a.space implies
        (affine_subspace_subset(affine_subspace_sup(a, b), c) =
            (affine_subspace_subset(a, c) and affine_subspace_subset(b, c)))
} by {
    if c.space = a.space {
        if affine_subspace_subset(affine_subspace_sup(a, b), c) {
            affine_subspace_subset_sup_left(a, b)
            affine_subspace_subset_trans(a, affine_subspace_sup(a, b), c)
            affine_subspace_subset(a, c)
            affine_subspace_subset_sup_right(a, b)
            affine_subspace_subset_trans(b, affine_subspace_sup(a, b), c)
            affine_subspace_subset(b, c)
            affine_subspace_subset(a, c) and affine_subspace_subset(b, c)
        }
        if affine_subspace_subset(a, c) and affine_subspace_subset(b, c) {
            affine_subspace_sup_subset_of_subset_left_right(a, b, c)
            affine_subspace_subset(affine_subspace_sup(a, b), c)
        }
        affine_subspace_subset(affine_subspace_sup(a, b), c) =
            (affine_subspace_subset(a, c) and affine_subspace_subset(b, c))
    }
}

/// A nonempty finite join is contained in every common upper bound over the
/// same ambient affine space.
theorem affine_subspace_list_sup_nonempty_subset_of_upper_bound[V: AddCommGroup, P](
    head: AffineSubspace[V, P],
    tail: List[AffineSubspace[V, P]],
    bound: AffineSubspace[V, P]
) {
    (forall(item: AffineSubspace[V, P]) {
        tail.contains(item) implies item.space = head.space
    }) and
    bound.space = head.space and
    affine_subspace_subset(head, bound) and
    (forall(item: AffineSubspace[V, P]) {
        tail.contains(item) implies affine_subspace_subset(item, bound)
    }) implies
        affine_subspace_subset(
            affine_subspace_list_sup_nonempty(head, tail), bound)
} by {
    define same_space(items: List[AffineSubspace[V, P]]) -> Bool {
        forall(item: AffineSubspace[V, P]) {
            items.contains(item) implies item.space = head.space
        }
    }
    define upper_bound(items: List[AffineSubspace[V, P]]) -> Bool {
        affine_subspace_subset(head, bound) and
        forall(item: AffineSubspace[V, P]) {
            items.contains(item) implies
                affine_subspace_subset(item, bound)
        }
    }
    define p(items: List[AffineSubspace[V, P]]) -> Bool {
        same_space(items) and
        bound.space = head.space and
        upper_bound(items) implies
            affine_subspace_subset(
                affine_subspace_list_sup_nonempty(head, items), bound)
    }

    if same_space(List.nil[AffineSubspace[V, P]]) and
        bound.space = head.space and
        upper_bound(List.nil[AffineSubspace[V, P]]) {
        upper_bound(List.nil[AffineSubspace[V, P]]) =
            (affine_subspace_subset(head, bound) and
                forall(item: AffineSubspace[V, P]) {
                    List.nil[AffineSubspace[V, P]].contains(item) implies
                        affine_subspace_subset(item, bound)
                })
        affine_subspace_subset(head, bound)
        affine_subspace_list_sup_nonempty_nil(head)
        affine_subspace_subset(
            affine_subspace_list_sup_nonempty(
                head, List.nil[AffineSubspace[V, P]]),
            bound)
    }
    p(List.nil[AffineSubspace[V, P]]) = (
        same_space(List.nil[AffineSubspace[V, P]]) and
        bound.space = head.space and
        upper_bound(List.nil[AffineSubspace[V, P]]) implies
            affine_subspace_subset(
                affine_subspace_list_sup_nonempty(
                    head, List.nil[AffineSubspace[V, P]]),
                bound))
    p(List.nil[AffineSubspace[V, P]])

    forall(item: AffineSubspace[V, P],
        items: List[AffineSubspace[V, P]]) {
        if p(items) {
            if same_space(List.cons(item, items)) and
                bound.space = head.space and
                upper_bound(List.cons(item, items)) {
                same_space(List.cons(item, items)) =
                    forall(candidate: AffineSubspace[V, P]) {
                        List.cons(item, items).contains(candidate) implies
                            candidate.space = head.space
                    }
                upper_bound(List.cons(item, items)) =
                    (affine_subspace_subset(head, bound) and
                        forall(candidate: AffineSubspace[V, P]) {
                            List.cons(item, items).contains(candidate) implies
                                affine_subspace_subset(candidate, bound)
                        })

                List.cons(item, items).contains(item)
                item.space = head.space
                bound.space = item.space
                affine_subspace_subset(item, bound)

                forall(candidate: AffineSubspace[V, P]) {
                    if items.contains(candidate) {
                        List.cons(item, items).contains(candidate)
                        candidate.space = head.space
                    }
                }
                forall(candidate: AffineSubspace[V, P]) {
                    if items.contains(candidate) {
                        List.cons(item, items).contains(candidate)
                        affine_subspace_subset(candidate, bound)
                    }
                }
                same_space(items) =
                    forall(candidate: AffineSubspace[V, P]) {
                        items.contains(candidate) implies
                            candidate.space = head.space
                    }
                same_space(items)
                upper_bound(items) =
                    (affine_subspace_subset(head, bound) and
                        forall(candidate: AffineSubspace[V, P]) {
                            items.contains(candidate) implies
                                affine_subspace_subset(candidate, bound)
                        })
                upper_bound(items)
                p(items) = (
                    same_space(items) and
                    bound.space = head.space and
                    upper_bound(items) implies
                        affine_subspace_subset(
                            affine_subspace_list_sup_nonempty(head, items),
                            bound))
                affine_subspace_subset(
                    affine_subspace_list_sup_nonempty(head, items),
                    bound)
                affine_subspace_sup_subset_of_subset_left_right(
                    item,
                    affine_subspace_list_sup_nonempty(head, items),
                    bound)
                affine_subspace_subset(
                    affine_subspace_sup(
                        item,
                        affine_subspace_list_sup_nonempty(head, items)),
                    bound)
                affine_subspace_list_sup_nonempty_cons(
                    head, item, items)
                affine_subspace_subset(
                    affine_subspace_list_sup_nonempty(
                        head, List.cons(item, items)),
                    bound)
            }
            p(List.cons(item, items)) = (
                same_space(List.cons(item, items)) and
                bound.space = head.space and
                upper_bound(List.cons(item, items)) implies
                    affine_subspace_subset(
                        affine_subspace_list_sup_nonempty(
                            head, List.cons(item, items)),
                        bound))
            p(List.cons(item, items))
        }
    }

    List.induction(function(items: List[AffineSubspace[V, P]]) {
        p(items)
    })
    forall(items: List[AffineSubspace[V, P]]) {
        p(items)
    }
    same_space(tail) = forall(item: AffineSubspace[V, P]) {
        tail.contains(item) implies item.space = head.space
    }
    upper_bound(tail) =
        (affine_subspace_subset(head, bound) and
            forall(item: AffineSubspace[V, P]) {
                tail.contains(item) implies
                    affine_subspace_subset(item, bound)
            })
    p(tail) = (
        same_space(tail) and
        bound.space = head.space and
        upper_bound(tail) implies
            affine_subspace_subset(
                affine_subspace_list_sup_nonempty(head, tail), bound))
    p(tail)
    if same_space(tail) and
        bound.space = head.space and
        upper_bound(tail) {
        affine_subspace_subset(
            affine_subspace_list_sup_nonempty(head, tail), bound)
    }
}

/// The join of two affine subspaces over a common ambient space is
/// commutative.
theorem affine_subspace_sup_comm[V: AddCommGroup, P](
    a: AffineSubspace[V, P],
    b: AffineSubspace[V, P]
) {
    a.space = b.space implies
        affine_subspace_sup(a, b) = affine_subspace_sup(b, a)
} by {
    if a.space = b.space {
        let left = affine_subspace_sup(a, b)
        let right = affine_subspace_sup(b, a)
        affine_subspace_sup_space(a, b)
        left.space = a.space
        affine_subspace_sup_space(b, a)
        right.space = b.space
        left.space = right.space

        affine_subspace_subset_sup_right(b, a)
        affine_subspace_subset(a, right)
        affine_subspace_subset_sup_left(b, a)
        affine_subspace_subset(b, right)
        affine_subspace_sup_subset_of_subset_left_right(a, b, right)
        affine_subspace_subset(left, right)

        affine_subspace_subset_sup_right(a, b)
        affine_subspace_subset(b, left)
        affine_subspace_subset_sup_left(a, b)
        affine_subspace_subset(a, left)
        affine_subspace_sup_subset_of_subset_left_right(b, a, left)
        affine_subspace_subset(right, left)

        affine_subspace_subset_antisymm(left, right)
        left = right
    }
}

/// Joining an affine subspace with itself gives the same affine subspace.
theorem affine_subspace_sup_idempotent[V: AddCommGroup, P](
    a: AffineSubspace[V, P]
) {
    affine_subspace_sup(a, a) = a
} by {
    affine_subspace_subset_refl(a)
    affine_subspace_sup_subset_of_subset_left_right(a, a, a)
    affine_subspace_subset(affine_subspace_sup(a, a), a)
    affine_subspace_subset_sup_left(a, a)
    affine_subspace_sup_space(a, a)
    affine_subspace_subset_antisymm(affine_subspace_sup(a, a), a)
    affine_subspace_sup(a, a) = a
}

/// Joining with a larger affine subspace on the right gives that larger
/// affine subspace.
theorem affine_subspace_sup_eq_right_of_subset[V: AddCommGroup, P](
    a: AffineSubspace[V, P],
    b: AffineSubspace[V, P]
) {
    a.space = b.space and affine_subspace_subset(a, b) implies
        affine_subspace_sup(a, b) = b
} by {
    if a.space = b.space and affine_subspace_subset(a, b) {
        b.space = a.space
        affine_subspace_subset_refl(b)
        affine_subspace_sup_subset_of_subset_left_right(a, b, b)
        affine_subspace_subset(affine_subspace_sup(a, b), b)
        affine_subspace_subset_sup_right(a, b)
        affine_subspace_sup_space(a, b)
        affine_subspace_sup(a, b).space = b.space
        affine_subspace_subset_antisymm(affine_subspace_sup(a, b), b)
        affine_subspace_sup(a, b) = b
    }
}

/// Joining with a larger affine subspace on the left gives that larger affine
/// subspace.
theorem affine_subspace_sup_eq_left_of_subset[V: AddCommGroup, P](
    a: AffineSubspace[V, P],
    b: AffineSubspace[V, P]
) {
    a.space = b.space and affine_subspace_subset(b, a) implies
        affine_subspace_sup(a, b) = a
} by {
    if a.space = b.space and affine_subspace_subset(b, a) {
        affine_subspace_subset_refl(a)
        affine_subspace_sup_subset_of_subset_left_right(a, b, a)
        affine_subspace_subset(affine_subspace_sup(a, b), a)
        affine_subspace_subset_sup_left(a, b)
        affine_subspace_sup_space(a, b)
        affine_subspace_subset_antisymm(affine_subspace_sup(a, b), a)
        affine_subspace_sup(a, b) = a
    }
}

/// An affine subspace is contained in another affine subspace over the same
/// ambient space exactly when their join is the latter subspace.
theorem affine_subspace_subset_iff_sup_eq_right[V: AddCommGroup, P](
    a: AffineSubspace[V, P],
    b: AffineSubspace[V, P]
) {
    a.space = b.space implies
        (affine_subspace_subset(a, b) =
            (affine_subspace_sup(a, b) = b))
} by {
    if a.space = b.space {
        if affine_subspace_subset(a, b) {
            affine_subspace_sup_eq_right_of_subset(a, b)
            affine_subspace_sup(a, b) = b
        }
        if affine_subspace_sup(a, b) = b {
            affine_subspace_subset_sup_left(a, b)
            affine_subspace_subset(a, affine_subspace_sup(a, b))
            affine_subspace_subset(a, b)
        }
        affine_subspace_subset(a, b) =
            (affine_subspace_sup(a, b) = b)
    }
}

/// An affine subspace contains another affine subspace over the same ambient
/// space exactly when their join is the former subspace.
theorem affine_subspace_subset_iff_sup_eq_left[V: AddCommGroup, P](
    a: AffineSubspace[V, P],
    b: AffineSubspace[V, P]
) {
    a.space = b.space implies
        (affine_subspace_subset(b, a) =
            (affine_subspace_sup(a, b) = a))
} by {
    if a.space = b.space {
        if affine_subspace_subset(b, a) {
            affine_subspace_sup_eq_left_of_subset(a, b)
            affine_subspace_sup(a, b) = a
        }
        if affine_subspace_sup(a, b) = a {
            affine_subspace_subset_sup_right(a, b)
            affine_subspace_subset(b, affine_subspace_sup(a, b))
            affine_subspace_subset(b, a)
        }
        affine_subspace_subset(b, a) =
            (affine_subspace_sup(a, b) = a)
    }
}

/// Two joins over a common ambient space are equal exactly when every summand
/// of either join is contained in the other join.
theorem affine_subspace_sup_eq_sup_iff[V: AddCommGroup, P](
    a: AffineSubspace[V, P],
    b: AffineSubspace[V, P],
    c: AffineSubspace[V, P],
    d: AffineSubspace[V, P]
) {
    a.space = b.space and a.space = c.space and a.space = d.space implies
        ((affine_subspace_sup(a, b) = affine_subspace_sup(c, d)) =
            ((affine_subspace_subset(a, affine_subspace_sup(c, d))
                and affine_subspace_subset(b, affine_subspace_sup(c, d)))
            and (affine_subspace_subset(c, affine_subspace_sup(a, b))
                and affine_subspace_subset(d, affine_subspace_sup(a, b)))))
} by {
    if a.space = b.space and a.space = c.space and a.space = d.space {
        let left = affine_subspace_sup(a, b)
        let right = affine_subspace_sup(c, d)
        affine_subspace_sup_space(a, b)
        left.space = a.space
        affine_subspace_sup_space(c, d)
        right.space = c.space
        left.space = right.space
        let condition = ((affine_subspace_subset(a, right)
                and affine_subspace_subset(b, right))
            and (affine_subspace_subset(c, left)
                and affine_subspace_subset(d, left)))

        if left = right {
            affine_subspace_subset_sup_left(a, b)
            affine_subspace_subset(a, left)
            affine_subspace_subset(a, right)
            affine_subspace_subset_sup_right(a, b)
            affine_subspace_subset(b, left)
            affine_subspace_subset(b, right)
            affine_subspace_subset_sup_left(c, d)
            affine_subspace_subset(c, right)
            affine_subspace_subset(c, left)
            affine_subspace_subset_sup_right(c, d)
            affine_subspace_subset(d, right)
            affine_subspace_subset(d, left)
            affine_subspace_subset(a, right) and
                affine_subspace_subset(b, right)
            affine_subspace_subset(c, left) and
                affine_subspace_subset(d, left)
            condition
        }

        if condition {
            affine_subspace_sup_subset_of_subset_left_right(a, b, right)
            affine_subspace_subset(left, right)
            affine_subspace_sup_subset_of_subset_left_right(c, d, left)
            affine_subspace_subset(right, left)
            affine_subspace_subset_antisymm(left, right)
            left = right
        }

        (left = right) = condition
        (affine_subspace_sup(a, b) = affine_subspace_sup(c, d)) =
            ((affine_subspace_subset(a, affine_subspace_sup(c, d))
                and affine_subspace_subset(b, affine_subspace_sup(c, d)))
            and (affine_subspace_subset(c, affine_subspace_sup(a, b))
                and affine_subspace_subset(d, affine_subspace_sup(a, b))))
    }
}

/// The join of the empty affine subspace with an affine subspace over the same
/// ambient space is that affine subspace.
theorem affine_subspace_empty_sup[V: AddCommGroup, P](
    a: AffineSpace[V, P],
    s: AffineSubspace[V, P]
) {
    s.space = a implies
        affine_subspace_sup(affine_subspace_empty(a), s) = s
} by {
    if s.space = a {
        AffineSubspace[V, P].new(a, affine_subspace_empty_contains[P]) =
            Option.some(affine_subspace_empty(a))
        affine_subspace_empty(a).space = a
        affine_subspace_empty(a).space = s.space
        affine_subspace_empty_subset(a, s)
        affine_subspace_sup_eq_right_of_subset(affine_subspace_empty(a), s)
        affine_subspace_sup(affine_subspace_empty(a), s) = s
    }
}

/// The join of an affine subspace with the empty affine subspace over the same
/// ambient space is that affine subspace.
theorem affine_subspace_sup_empty[V: AddCommGroup, P](
    a: AffineSpace[V, P],
    s: AffineSubspace[V, P]
) {
    s.space = a implies
        affine_subspace_sup(s, affine_subspace_empty(a)) = s
} by {
    if s.space = a {
        AffineSubspace[V, P].new(a, affine_subspace_empty_contains[P]) =
            Option.some(affine_subspace_empty(a))
        affine_subspace_empty(a).space = a
        s.space = affine_subspace_empty(a).space
        affine_subspace_empty_subset(a, s)
        affine_subspace_sup_eq_left_of_subset(s, affine_subspace_empty(a))
        affine_subspace_sup(s, affine_subspace_empty(a)) = s
    }
}

/// The join of an affine subspace with the full affine subspace over the same
/// ambient space is the full affine subspace.
theorem affine_subspace_sup_univ[V: AddCommGroup, P](
    a: AffineSpace[V, P],
    s: AffineSubspace[V, P]
) {
    s.space = a implies
        affine_subspace_sup(s, affine_subspace_univ(a)) =
            affine_subspace_univ(a)
} by {
    if s.space = a {
        AffineSubspace[V, P].new(a, affine_subspace_univ_contains[P]) =
            Option.some(affine_subspace_univ(a))
        affine_subspace_univ(a).space = a
        s.space = affine_subspace_univ(a).space
        affine_subspace_subset_univ(a, s)
        affine_subspace_sup_eq_right_of_subset(s, affine_subspace_univ(a))
        affine_subspace_sup(s, affine_subspace_univ(a)) =
            affine_subspace_univ(a)
    }
}

/// The join of the full affine subspace with an affine subspace over the same
/// ambient space is the full affine subspace.
theorem affine_subspace_univ_sup[V: AddCommGroup, P](
    a: AffineSpace[V, P],
    s: AffineSubspace[V, P]
) {
    s.space = a implies
        affine_subspace_sup(affine_subspace_univ(a), s) =
            affine_subspace_univ(a)
} by {
    if s.space = a {
        AffineSubspace[V, P].new(a, affine_subspace_univ_contains[P]) =
            Option.some(affine_subspace_univ(a))
        affine_subspace_univ(a).space = a
        affine_subspace_univ(a).space = s.space
        affine_subspace_subset_univ(a, s)
        affine_subspace_sup_eq_left_of_subset(affine_subspace_univ(a), s)
    }
}

/// Affine-subspace joins over a common ambient space are monotone in the left
/// argument.
theorem affine_subspace_sup_mono_left[V: AddCommGroup, P](
    a: AffineSubspace[V, P],
    b: AffineSubspace[V, P],
    c: AffineSubspace[V, P]
) {
    a.space = b.space and a.space = c.space
        and affine_subspace_subset(a, b) implies
        affine_subspace_subset(
            affine_subspace_sup(a, c), affine_subspace_sup(b, c))
} by {
    if a.space = b.space and a.space = c.space
        and affine_subspace_subset(a, b) {
        let target = affine_subspace_sup(b, c)
        affine_subspace_sup_space(b, c)
        target.space = a.space
        affine_subspace_subset_sup_left(b, c)
        affine_subspace_subset(b, target)
        affine_subspace_subset_trans(a, b, target)
        affine_subspace_subset(a, target)
        affine_subspace_subset_sup_right(b, c)
        affine_subspace_subset(c, target)
        affine_subspace_sup_subset_of_subset_left_right(a, c, target)
        affine_subspace_subset(affine_subspace_sup(a, c), target)
        affine_subspace_subset(
            affine_subspace_sup(a, c), affine_subspace_sup(b, c))
    }
}

/// Affine-subspace joins over a common ambient space are monotone in the right
/// argument.
theorem affine_subspace_sup_mono_right[V: AddCommGroup, P](
    a: AffineSubspace[V, P],
    b: AffineSubspace[V, P],
    c: AffineSubspace[V, P]
) {
    a.space = b.space and a.space = c.space
        and affine_subspace_subset(a, b) implies
        affine_subspace_subset(
            affine_subspace_sup(c, a), affine_subspace_sup(c, b))
} by {
    if a.space = b.space and a.space = c.space
        and affine_subspace_subset(a, b) {
        let target = affine_subspace_sup(c, b)
        affine_subspace_sup_space(c, b)
        target.space = c.space
        affine_subspace_subset_sup_left(c, b)
        affine_subspace_subset(c, target)
        affine_subspace_subset_sup_right(c, b)
        affine_subspace_subset(b, target)
        affine_subspace_subset_trans(a, b, target)
        affine_subspace_subset(a, target)
        affine_subspace_sup_subset_of_subset_left_right(c, a, target)
        affine_subspace_subset(affine_subspace_sup(c, a), target)
        affine_subspace_subset(
            affine_subspace_sup(c, a), affine_subspace_sup(c, b))
    }
}

/// Affine-subspace joins over a common ambient space are monotone in both
/// arguments.
theorem affine_subspace_sup_mono[V: AddCommGroup, P](
    a: AffineSubspace[V, P],
    b: AffineSubspace[V, P],
    c: AffineSubspace[V, P],
    d: AffineSubspace[V, P]
) {
    a.space = b.space and a.space = c.space and a.space = d.space
        and affine_subspace_subset(a, b) and affine_subspace_subset(c, d)
        implies affine_subspace_subset(
            affine_subspace_sup(a, c), affine_subspace_sup(b, d))
} by {
    if a.space = b.space and a.space = c.space and a.space = d.space
        and affine_subspace_subset(a, b) and affine_subspace_subset(c, d) {
        let target = affine_subspace_sup(b, d)
        affine_subspace_sup_space(b, d)
        target.space = a.space
        affine_subspace_subset_sup_left(b, d)
        affine_subspace_subset(b, target)
        affine_subspace_subset_trans(a, b, target)
        affine_subspace_subset(a, target)
        affine_subspace_subset_sup_right(b, d)
        affine_subspace_subset(d, target)
        affine_subspace_subset_trans(c, d, target)
        affine_subspace_subset(c, target)
        affine_subspace_sup_subset_of_subset_left_right(a, c, target)
        affine_subspace_subset(affine_subspace_sup(a, c), target)
        affine_subspace_subset(
            affine_subspace_sup(a, c), affine_subspace_sup(b, d))
    }
}

/// The join of three affine subspaces over a common ambient space is
/// associative.
theorem affine_subspace_sup_assoc[V: AddCommGroup, P](
    a: AffineSubspace[V, P],
    b: AffineSubspace[V, P],
    c: AffineSubspace[V, P]
) {
    a.space = b.space and a.space = c.space implies
        affine_subspace_sup(affine_subspace_sup(a, b), c) =
            affine_subspace_sup(a, affine_subspace_sup(b, c))
} by {
    if a.space = b.space and a.space = c.space {
        let ab = affine_subspace_sup(a, b)
        let bc = affine_subspace_sup(b, c)
        let left = affine_subspace_sup(ab, c)
        let right = affine_subspace_sup(a, bc)
        affine_subspace_sup_space(a, b)
        ab.space = a.space
        affine_subspace_sup_space(b, c)
        bc.space = b.space
        affine_subspace_sup_space(ab, c)
        left.space = ab.space
        affine_subspace_sup_space(a, bc)
        right.space = a.space
        left.space = right.space

        affine_subspace_subset_sup_left(a, bc)
        affine_subspace_subset(a, right)
        affine_subspace_subset_sup_left(b, c)
        affine_subspace_subset(b, bc)
        affine_subspace_subset_sup_right(a, bc)
        affine_subspace_subset(bc, right)
        affine_subspace_subset_trans(b, bc, right)
        affine_subspace_subset(b, right)
        affine_subspace_sup_subset_of_subset_left_right(a, b, right)
        affine_subspace_subset(ab, right)
        affine_subspace_subset_sup_right(b, c)
        affine_subspace_subset(c, bc)
        affine_subspace_subset_trans(c, bc, right)
        affine_subspace_subset(c, right)
        affine_subspace_sup_subset_of_subset_left_right(ab, c, right)
        affine_subspace_subset(left, right)

        affine_subspace_subset_sup_left(a, b)
        affine_subspace_subset(a, ab)
        affine_subspace_subset_sup_left(ab, c)
        affine_subspace_subset(ab, left)
        affine_subspace_subset_trans(a, ab, left)
        affine_subspace_subset(a, left)
        affine_subspace_subset_sup_right(a, b)
        affine_subspace_subset(b, ab)
        affine_subspace_subset_trans(b, ab, left)
        affine_subspace_subset(b, left)
        affine_subspace_subset_sup_right(ab, c)
        affine_subspace_subset(c, left)
        affine_subspace_sup_subset_of_subset_left_right(b, c, left)
        affine_subspace_subset(bc, left)
        affine_subspace_sup_subset_of_subset_left_right(a, bc, left)
        affine_subspace_subset(right, left)

        affine_subspace_subset_antisymm(left, right)
        left = right
        affine_subspace_sup(affine_subspace_sup(a, b), c) =
            affine_subspace_sup(a, affine_subspace_sup(b, c))
    }
}

/// The affine span of an empty source set is contained in every affine subspace
/// over the same ambient space.
theorem affine_span_empty_subset[V: AddCommGroup, P](
    a: AffineSpace[V, P], t: AffineSubspace[V, P]) {
    t.space = a implies
        affine_subspace_subset(affine_span(a, affine_subspace_empty_contains[P]), t)
} by {
    if t.space = a {
        affine_span_subset(a, affine_subspace_empty_contains[P], t)
    }
}

/// The affine span of the full predicate equals the full affine subspace.
theorem affine_span_univ_subset[V: AddCommGroup, P](
    a: AffineSpace[V, P]) {
    affine_subspace_subset(affine_subspace_univ(a),
        affine_span(a, affine_subspace_univ_contains[P]))
} by {
    forall(x: P) {
        if affine_subspace_univ(a).contains(x) {
            affine_span_contains_src(a, affine_subspace_univ_contains[P], x)
            affine_span(a, affine_subspace_univ_contains[P]).contains(x)
        }
    }
}

/// The affine span of the empty source set equals the empty affine subspace.
theorem affine_span_empty_eq[V: AddCommGroup, P](a: AffineSpace[V, P]) {
    affine_span(a, affine_subspace_empty_contains[P]) = affine_subspace_empty(a)
} by {
    let lhs = affine_span(a, affine_subspace_empty_contains[P])
    let rhs = affine_subspace_empty(a)
    affine_span_space(a, affine_subspace_empty_contains[P])
    lhs.space = a
    AffineSubspace[V, P].new(a, affine_subspace_empty_contains[P]) = Option.some(rhs)
    rhs.space = a
    affine_subspace_empty_subset(a, rhs)
    affine_subspace_subset(lhs, rhs)
    affine_subspace_subset(lhs, rhs) = forall(y: P) {
        lhs.contains(y) implies rhs.contains(y)
    }
    forall(x: P) {
        if lhs.contains(x) {
            affine_subspace_empty_not_contains(a, x)
            false
        }
        if rhs.contains(x) {
            affine_subspace_empty_not_contains(a, x)
            false
        }
        lhs.contains(x) = rhs.contains(x)
    }
    affine_subspace_ext(lhs, rhs)
}

/// The source predicate consisting of the points in a finite list.
define affine_span_list_src[P](points: List[P], x: P) -> Bool {
    points.contains(x)
}

/// Membership in a finite-list source is membership in its list of points.
theorem affine_span_list_src_iff[P](points: List[P], x: P) {
    affine_span_list_src(points, x) = points.contains(x)
}

/// The empty list has the empty affine-span source predicate.
theorem affine_span_list_src_nil[P](x: P) {
    affine_span_list_src(List.nil[P], x) = false
}

/// Membership in a source list with a new head is equality with the head or
/// membership in the tail.
theorem affine_span_list_src_cons[P](
    head: P, tail: List[P], x: P
) {
    affine_span_list_src(List.cons(head, tail), x) =
        (head = x or affine_span_list_src(tail, x))
} by {
    affine_span_list_src_iff(List.cons(head, tail), x)
    affine_span_list_src_iff(tail, x)
    if head = x {
        List.cons(head, tail).contains(x) = true
        (head = x or affine_span_list_src(tail, x)) = true
        affine_span_list_src(List.cons(head, tail), x) =
            (head = x or affine_span_list_src(tail, x))
    } else {
        if List.cons(head, tail).contains(x) {
            tail.contains(x)
        }
        if tail.contains(x) {
            List.cons(head, tail).contains(x)
        }
        List.cons(head, tail).contains(x) = tail.contains(x)
        (head = x or affine_span_list_src(tail, x)) =
            affine_span_list_src(tail, x)
        affine_span_list_src(List.cons(head, tail), x) =
            (head = x or affine_span_list_src(tail, x))
    }
}

/// The source predicate of a concatenated list is the union of the source
/// predicates of its parts.
theorem affine_span_list_src_add_eq_union[P](
    left: List[P], right: List[P], x: P
) {
    affine_span_list_src(left + right, x) =
        affine_span_union_src(
            affine_span_list_src[P](left),
            affine_span_list_src[P](right), x)
} by {
    affine_span_list_src_iff(left + right, x)
    affine_span_list_src_iff(left, x)
    affine_span_list_src_iff(right, x)
    affine_span_union_src_iff(
        affine_span_list_src[P](left),
        affine_span_list_src[P](right), x)
    if (left + right).contains(x) {
        add_contains_or(left, right, x)
        if left.contains(x) {
            affine_span_list_src(left, x)
        } else {
            right.contains(x)
            affine_span_list_src(right, x)
        }
        affine_span_union_src(
            affine_span_list_src[P](left),
            affine_span_list_src[P](right), x)
    }
    if affine_span_union_src(
        affine_span_list_src[P](left),
        affine_span_list_src[P](right), x) {
        affine_span_union_src_iff(
            affine_span_list_src[P](left),
            affine_span_list_src[P](right), x)
        if affine_span_list_src(left, x) {
            left.contains(x)
            add_contains_left(left, right, x)
        } else {
            affine_span_list_src(right, x)
            right.contains(x)
            add_contains_right(left, right, x)
        }
        (left + right).contains(x)
    }
}

/// The affine span of the empty list is the empty affine subspace.
theorem affine_span_list_nil_eq[V: AddCommGroup, P](
    a: AffineSpace[V, P]
) {
    affine_span(a, affine_span_list_src[P](List.nil[P])) =
        affine_subspace_empty(a)
} by {
    forall(x: P) {
        affine_span_list_src_nil(x)
        affine_subspace_empty_contains[P](x) = false
        affine_span_list_src(List.nil[P], x) =
            affine_subspace_empty_contains[P](x)
    }
    affine_span_congr(
        a, affine_span_list_src[P](List.nil[P]),
        affine_subspace_empty_contains[P])
    affine_span(a, affine_span_list_src[P](List.nil[P])) =
        affine_span(a, affine_subspace_empty_contains[P])
    affine_span_empty_eq(a)
}

/// The head of a nonempty list belongs to its affine span.
theorem affine_span_list_cons_contains_head[V: AddCommGroup, P](
    a: AffineSpace[V, P], head: P, tail: List[P]
) {
    affine_span(
        a, affine_span_list_src[P](List.cons(head, tail))).contains(head)
} by {
    affine_span_list_src_cons(head, tail, head)
    affine_span_list_src(List.cons(head, tail), head)
    affine_span_contains_src(
        a, affine_span_list_src[P](List.cons(head, tail)), head)
}

/// The affine span of a list tail is contained in the affine span after
/// inserting a new head.
theorem affine_span_list_tail_subset_cons[V: AddCommGroup, P](
    a: AffineSpace[V, P], head: P, tail: List[P]
) {
    affine_subspace_subset(
        affine_span(a, affine_span_list_src[P](tail)),
        affine_span(a, affine_span_list_src[P](List.cons(head, tail))))
} by {
    forall(x: P) {
        if affine_span_list_src(tail, x) {
            affine_span_list_src_cons(head, tail, x)
            affine_span_list_src(List.cons(head, tail), x)
        }
    }
    affine_span_mono(
        a, affine_span_list_src[P](tail),
        affine_span_list_src[P](List.cons(head, tail)))
}

/// A list head already belonging to the affine span of the tail can be
/// deleted without changing the affine span.
theorem affine_span_list_cons_eq_tail_of_contains[V: AddCommGroup, P](
    a: AffineSpace[V, P], head: P, tail: List[P]
) {
    affine_span(a, affine_span_list_src[P](tail)).contains(head) implies
        affine_span(a, affine_span_list_src[P](List.cons(head, tail))) =
            affine_span(a, affine_span_list_src[P](tail))
} by {
    if affine_span(a, affine_span_list_src[P](tail)).contains(head) {
        let tail_span = affine_span(a, affine_span_list_src[P](tail))
        let cons_span =
            affine_span(a, affine_span_list_src[P](List.cons(head, tail)))
        affine_span_space(a, affine_span_list_src[P](tail))
        tail_span.space = a
        affine_span_space(
            a, affine_span_list_src[P](List.cons(head, tail)))
        cons_span.space = a
        affine_span_list_tail_subset_cons(a, head, tail)
        affine_subspace_subset(tail_span, cons_span)

        forall(x: P) {
            if affine_span_list_src(List.cons(head, tail), x) {
                affine_span_list_src_cons(head, tail, x)
                if head = x {
                    tail_span.contains(x) = tail_span.contains(head)
                    tail_span.contains(x)
                } else {
                    affine_span_list_src(tail, x)
                    affine_span_contains_src(
                        a, affine_span_list_src[P](tail), x)
                    tail_span.contains(x)
                }
            }
        }
        affine_span_subset(
            a, affine_span_list_src[P](List.cons(head, tail)), tail_span)
        affine_subspace_subset(cons_span, tail_span)

        cons_span.space = tail_span.space
        affine_subspace_subset_antisymm(cons_span, tail_span)
        cons_span = tail_span
        affine_span(a, affine_span_list_src[P](List.cons(head, tail))) =
            affine_span(a, affine_span_list_src[P](tail))
    }
}

/// The membership predicate of the source set consisting of just the point `p`.
define affine_span_singleton_src[P](p: P, x: P) -> Bool {
    x = p
}

/// The affine span of a single point is the singleton affine subspace at that point.
theorem affine_span_singleton_eq[V: AddCommGroup, P](a: AffineSpace[V, P], p: P) {
    affine_span(a, affine_span_singleton_src[P](p)) = affine_subspace_singleton(a, p)
} by {
    let src = affine_span_singleton_src[P](p)
    let lhs = affine_span(a, src)
    let rhs = affine_subspace_singleton(a, p)
    affine_span_space(a, src)
    lhs.space = a
    affine_subspace_singleton_space(a, p)
    rhs.space = a
    affine_subspace_singleton_contains_self(a, p)
    rhs.contains(p)
    forall(y: P) {
        if src(y) {
            rhs.contains(y)
        }
    }
    affine_span_subset(a, src, rhs)
    affine_subspace_subset(lhs, rhs)
    affine_subspace_subset(lhs, rhs) = forall(z: P) {
        lhs.contains(z) implies rhs.contains(z)
    }
    affine_span_contains_src(a, src, p)
    src(p) = (p = p)
    src(p)
    lhs.contains(p)
    forall(x: P) {
        if lhs.contains(x) {
            rhs.contains(x)
        }
        if rhs.contains(x) {
            affine_subspace_singleton_contains_iff(a, p, x)
            lhs.contains(x) = lhs.contains(p)
            lhs.contains(x)
        }
        lhs.contains(x) = rhs.contains(x)
    }
    affine_subspace_ext(lhs, rhs)
}

/// The affine span of a singleton list is the singleton affine subspace at its
/// point.
theorem affine_span_list_singleton_eq[V: AddCommGroup, P](
    a: AffineSpace[V, P], p: P
) {
    affine_span(a, affine_span_list_src[P](List.singleton(p))) =
        affine_subspace_singleton(a, p)
} by {
    forall(x: P) {
        affine_span_list_src_cons(p, List.nil[P], x)
        affine_span_list_src_nil(x)
        if p = x {
            x = p
        }
        if x = p {
            p = x
        }
        affine_span_list_src(List.singleton(p), x) = (p = x)
        affine_span_singleton_src(p, x) = (x = p)
        (p = x) = (x = p)
        affine_span_list_src(List.singleton(p), x) =
            affine_span_singleton_src(p, x)
    }
    affine_span_congr(
        a, affine_span_list_src[P](List.singleton(p)),
        affine_span_singleton_src[P](p))
    affine_span(a, affine_span_list_src[P](List.singleton(p))) =
        affine_span(a, affine_span_singleton_src[P](p))
    affine_span_singleton_eq(a, p)
}

/// Permuting a finite list of points does not change its source predicate.
theorem affine_span_list_src_permutation_eq[P](
    left: List[P], right: List[P]
) {
    is_permutation(left, right) implies
        forall(x: P) {
            affine_span_list_src(left, x) =
                affine_span_list_src(right, x)
        }
} by {
    if is_permutation(left, right) {
        forall(x: P) {
            if left.contains(x) {
                list_contains_implies_count_geq_one(left, x)
                if not right.contains(x) {
                    list_not_contains_impl_count_zero(right, x)
                    left.count(x) = right.count(x)
                    false
                }
                right.contains(x)
            }
            if right.contains(x) {
                list_contains_implies_count_geq_one(right, x)
                if not left.contains(x) {
                    list_not_contains_impl_count_zero(left, x)
                    left.count(x) = right.count(x)
                    false
                }
                left.contains(x)
            }
            left.contains(x) = right.contains(x)
            affine_span_list_src_iff(left, x)
            affine_span_list_src_iff(right, x)
            affine_span_list_src(left, x) =
                affine_span_list_src(right, x)
        }
    }
}

/// Permuting a finite list of points does not change its affine span.
theorem affine_span_list_permutation_eq[V: AddCommGroup, P](
    a: AffineSpace[V, P], left: List[P], right: List[P]
) {
    is_permutation(left, right) implies
        affine_span(a, affine_span_list_src[P](left)) =
            affine_span(a, affine_span_list_src[P](right))
} by {
    if is_permutation(left, right) {
        affine_span_list_src_permutation_eq(left, right)
        affine_span_congr(
            a, affine_span_list_src[P](left),
            affine_span_list_src[P](right))
    }
}

/// Removing duplicate points from a finite list does not change its affine
/// span.
theorem affine_span_list_unique_eq[V: AddCommGroup, P](
    a: AffineSpace[V, P], points: List[P]
) {
    affine_span(a, affine_span_list_src[P](points.unique)) =
        affine_span(a, affine_span_list_src[P](points))
} by {
    forall(x: P) {
        affine_span_list_src_iff(points.unique, x)
        affine_span_list_src_iff(points, x)
        unique_preserves_contains(points, x)
        affine_span_list_src(points.unique, x) =
            affine_span_list_src(points, x)
    }
    affine_span_congr(
        a, affine_span_list_src[P](points.unique),
        affine_span_list_src[P](points))
}

/// Inclusion between the points of finite lists induces inclusion between
/// their source predicates.
theorem affine_span_list_src_mono[P](
    left: List[P], right: List[P]
) {
    (forall(x: P) { left.contains(x) implies right.contains(x) }) implies
        forall(x: P) {
            affine_span_list_src(left, x) implies
                affine_span_list_src(right, x)
        }
} by {
    if forall(x: P) { left.contains(x) implies right.contains(x) } {
        forall(x: P) {
            if affine_span_list_src(left, x) {
                affine_span_list_src_iff(left, x)
                left.contains(x)
                right.contains(x)
                affine_span_list_src_iff(right, x)
                affine_span_list_src(right, x)
            }
        }
    }
}

/// Inclusion between the points of finite lists induces inclusion between
/// their affine spans.
theorem affine_span_list_mono[V: AddCommGroup, P](
    a: AffineSpace[V, P], left: List[P], right: List[P]
) {
    (forall(x: P) { left.contains(x) implies right.contains(x) }) implies
        affine_subspace_subset(
            affine_span(a, affine_span_list_src[P](left)),
            affine_span(a, affine_span_list_src[P](right)))
} by {
    if forall(x: P) { left.contains(x) implies right.contains(x) } {
        let left_src = affine_span_list_src[P](left)
        let right_src = affine_span_list_src[P](right)
        forall(x: P) {
            if left_src(x) {
                left_src(x) = affine_span_list_src(left, x)
                affine_span_list_src_iff(left, x)
                left.contains(x)
                right.contains(x)
                affine_span_list_src_iff(right, x)
                right_src(x) = affine_span_list_src(right, x)
                right_src(x)
            }
        }
        affine_span_mono(a, left_src, right_src)
        affine_subspace_subset(
            affine_span(a, left_src), affine_span(a, right_src))
        left_src = affine_span_list_src[P](left)
        right_src = affine_span_list_src[P](right)
        affine_subspace_subset(
            affine_span(a, affine_span_list_src[P](left)),
            affine_span(a, affine_span_list_src[P](right)))
    }
}

/// Finite lists with the same points have the same affine span.
theorem affine_span_list_eq_of_mutual_contains[V: AddCommGroup, P](
    a: AffineSpace[V, P], left: List[P], right: List[P]
) {
    ((forall(x: P) { left.contains(x) implies right.contains(x) })
        and (forall(x: P) { right.contains(x) implies left.contains(x) }))
        implies
            affine_span(a, affine_span_list_src[P](left)) =
                affine_span(a, affine_span_list_src[P](right))
} by {
    if (forall(x: P) { left.contains(x) implies right.contains(x) })
        and (forall(x: P) { right.contains(x) implies left.contains(x) }) {
        let left_span = affine_span(a, affine_span_list_src[P](left))
        let right_span = affine_span(a, affine_span_list_src[P](right))

        affine_span_list_mono(a, left, right)
        affine_subspace_subset(left_span, right_span)
        affine_span_list_mono(a, right, left)
        affine_subspace_subset(right_span, left_span)

        affine_span_space(a, affine_span_list_src[P](left))
        left_span.space = a
        affine_span_space(a, affine_span_list_src[P](right))
        right_span.space = a
        left_span.space = right_span.space
        affine_subspace_subset_antisymm(left_span, right_span)
        left_span = right_span
        left_span = affine_span(a, affine_span_list_src[P](left))
        right_span = affine_span(a, affine_span_list_src[P](right))
        affine_span(a, affine_span_list_src[P](left)) =
            affine_span(a, affine_span_list_src[P](right))
    }
}

/// A finite-list affine span is contained in an affine subspace exactly when
/// every point in the list belongs to that subspace.
theorem affine_span_list_subset_iff[V: AddCommGroup, P](
    a: AffineSpace[V, P],
    points: List[P],
    t: AffineSubspace[V, P]
) {
    t.space = a implies
        (affine_subspace_subset(
            affine_span(a, affine_span_list_src[P](points)), t) =
        forall(x: P) { points.contains(x) implies t.contains(x) })
} by {
    if t.space = a {
        affine_span_subset_iff(
            a, affine_span_list_src[P](points), t)
        affine_subspace_subset(
            affine_span(a, affine_span_list_src[P](points)), t) =
            forall(x: P) {
                affine_span_list_src(points, x) implies t.contains(x)
            }
        forall(x: P) {
            affine_span_list_src_iff(points, x)
        }
        (forall(x: P) {
            affine_span_list_src(points, x) implies t.contains(x)
        }) = (forall(x: P) {
            points.contains(x) implies t.contains(x)
        })
        affine_subspace_subset(
            affine_span(a, affine_span_list_src[P](points)), t) =
            forall(x: P) { points.contains(x) implies t.contains(x) }
    }
}

/// A finite-list affine span equals a target affine subspace exactly when the
/// target contains every listed point and is contained in their affine span.
theorem affine_span_list_eq_target_iff[V: AddCommGroup, P](
    a: AffineSpace[V, P],
    points: List[P],
    t: AffineSubspace[V, P]
) {
    t.space = a implies
        ((affine_span(a, affine_span_list_src[P](points)) = t) =
        ((forall(x: P) { points.contains(x) implies t.contains(x) })
        and affine_subspace_subset(
            t, affine_span(a, affine_span_list_src[P](points)))))
} by {
    if t.space = a {
        let span = affine_span(a, affine_span_list_src[P](points))
        let generators_contained =
            forall(x: P) { points.contains(x) implies t.contains(x) }
        let target_contained = affine_subspace_subset(t, span)
        let condition = generators_contained and target_contained

        if span = t {
            forall(x: P) {
                if points.contains(x) {
                    affine_span_list_src_iff(points, x)
                    affine_span_list_src(points, x)
                    affine_span_contains_src(
                        a, affine_span_list_src[P](points), x)
                    span.contains(x)
                    t.contains(x)
                }
            }
            generators_contained
            affine_subspace_subset_refl(t)
            target_contained
            condition
        }

        if condition {
            generators_contained
            target_contained
            affine_span_list_subset_iff(a, points, t)
            affine_subspace_subset(span, t)
            affine_span_space(a, affine_span_list_src[P](points))
            span.space = a
            span.space = t.space
            affine_subspace_subset_antisymm(span, t)
            span = t
        }

        (span = t) = condition
        span = affine_span(a, affine_span_list_src[P](points))
        generators_contained =
            forall(x: P) { points.contains(x) implies t.contains(x) }
        target_contained = affine_subspace_subset(
            t, affine_span(a, affine_span_list_src[P](points)))
        condition =
            ((forall(x: P) { points.contains(x) implies t.contains(x) })
            and affine_subspace_subset(
                t, affine_span(a, affine_span_list_src[P](points))))
        (affine_span(a, affine_span_list_src[P](points)) = t) =
            ((forall(x: P) { points.contains(x) implies t.contains(x) })
            and affine_subspace_subset(
                t, affine_span(a, affine_span_list_src[P](points))))
    }
}

/// The affine span of a concatenated list is the join of the affine spans of
/// its two parts.
theorem affine_span_list_add_eq_sup[V: AddCommGroup, P](
    a: AffineSpace[V, P], left: List[P], right: List[P]
) {
    affine_span(a, affine_span_list_src[P](left + right)) =
        affine_subspace_sup(
            affine_span(a, affine_span_list_src[P](left)),
            affine_span(a, affine_span_list_src[P](right)))
} by {
    forall(x: P) {
        affine_span_list_src_add_eq_union(left, right, x)
    }
    affine_span_congr(
        a, affine_span_list_src[P](left + right),
        affine_span_union_src(
            affine_span_list_src[P](left),
            affine_span_list_src[P](right)))
    affine_span(a, affine_span_list_src[P](left + right)) =
        affine_span(
            a, affine_span_union_src(
                affine_span_list_src[P](left),
                affine_span_list_src[P](right)))
    affine_subspace_sup_affine_spans_eq_span_union(
        a, affine_span_list_src[P](left),
        affine_span_list_src[P](right))
}

/// Replacing the right part of a concatenated list by a list with the same
/// affine span does not change the affine span of the concatenation.
theorem affine_span_list_add_right_congr[V: AddCommGroup, P](
    a: AffineSpace[V, P],
    left: List[P],
    right: List[P],
    replacement: List[P]
) {
    affine_span(a, affine_span_list_src[P](right)) =
        affine_span(a, affine_span_list_src[P](replacement)) implies
    affine_span(a, affine_span_list_src[P](left + right)) =
        affine_span(a, affine_span_list_src[P](left + replacement))
} by {
    if affine_span(a, affine_span_list_src[P](right)) =
        affine_span(a, affine_span_list_src[P](replacement)) {
        affine_span_list_add_eq_sup(a, left, right)
        affine_span_list_add_eq_sup(a, left, replacement)
        affine_subspace_sup(
            affine_span(a, affine_span_list_src[P](left)),
            affine_span(a, affine_span_list_src[P](right))) =
        affine_subspace_sup(
            affine_span(a, affine_span_list_src[P](left)),
            affine_span(a, affine_span_list_src[P](replacement)))
        affine_span(a, affine_span_list_src[P](left + right)) =
            affine_span(a, affine_span_list_src[P](left + replacement))
    }
}

/// Replacing the left part of a concatenated list by a list with the same
/// affine span does not change the affine span of the concatenation.
theorem affine_span_list_add_left_congr[V: AddCommGroup, P](
    a: AffineSpace[V, P],
    left: List[P],
    replacement: List[P],
    right: List[P]
) {
    affine_span(a, affine_span_list_src[P](left)) =
        affine_span(a, affine_span_list_src[P](replacement)) implies
    affine_span(a, affine_span_list_src[P](left + right)) =
        affine_span(a, affine_span_list_src[P](replacement + right))
} by {
    if affine_span(a, affine_span_list_src[P](left)) =
        affine_span(a, affine_span_list_src[P](replacement)) {
        affine_span_list_add_eq_sup(a, left, right)
        affine_span_list_add_eq_sup(a, replacement, right)
        affine_subspace_sup(
            affine_span(a, affine_span_list_src[P](left)),
            affine_span(a, affine_span_list_src[P](right))) =
        affine_subspace_sup(
            affine_span(a, affine_span_list_src[P](replacement)),
            affine_span(a, affine_span_list_src[P](right)))
        affine_span(a, affine_span_list_src[P](left + right)) =
            affine_span(a, affine_span_list_src[P](replacement + right))
    }
}

/// Replacing both parts of a concatenated list by lists with the same
/// respective affine spans does not change the affine span of the
/// concatenation.
theorem affine_span_list_add_congr[V: AddCommGroup, P](
    a: AffineSpace[V, P],
    left: List[P],
    left_replacement: List[P],
    right: List[P],
    right_replacement: List[P]
) {
    (affine_span(a, affine_span_list_src[P](left)) =
        affine_span(a, affine_span_list_src[P](left_replacement))
    and affine_span(a, affine_span_list_src[P](right)) =
        affine_span(a, affine_span_list_src[P](right_replacement))) implies
    affine_span(a, affine_span_list_src[P](left + right)) =
        affine_span(
            a, affine_span_list_src[P](
                left_replacement + right_replacement))
} by {
    if affine_span(a, affine_span_list_src[P](left)) =
        affine_span(a, affine_span_list_src[P](left_replacement))
    and affine_span(a, affine_span_list_src[P](right)) =
        affine_span(a, affine_span_list_src[P](right_replacement)) {
        affine_span_list_add_left_congr(
            a, left, left_replacement, right)
        affine_span(a, affine_span_list_src[P](left + right)) =
            affine_span(
                a, affine_span_list_src[P](left_replacement + right))
        affine_span_list_add_right_congr(
            a, left_replacement, right, right_replacement)
        affine_span(
            a, affine_span_list_src[P](left_replacement + right)) =
            affine_span(
                a, affine_span_list_src[P](
                    left_replacement + right_replacement))
        affine_span(a, affine_span_list_src[P](left + right)) =
            affine_span(
                a, affine_span_list_src[P](
                    left_replacement + right_replacement))
    }
}

/// Swapping the two parts of a concatenated list does not change its affine
/// span.
theorem affine_span_list_add_comm[V: AddCommGroup, P](
    a: AffineSpace[V, P], left: List[P], right: List[P]
) {
    affine_span(a, affine_span_list_src[P](left + right)) =
        affine_span(a, affine_span_list_src[P](right + left))
} by {
    let left_span = affine_span(a, affine_span_list_src[P](left))
    let right_span = affine_span(a, affine_span_list_src[P](right))

    affine_span_space(a, affine_span_list_src[P](left))
    left_span.space = a
    affine_span_space(a, affine_span_list_src[P](right))
    right_span.space = a
    left_span.space = right_span.space
    affine_subspace_sup_comm(left_span, right_span)
    affine_subspace_sup(left_span, right_span) =
        affine_subspace_sup(right_span, left_span)

    affine_span_list_add_eq_sup(a, left, right)
    affine_span(a, affine_span_list_src[P](left + right)) =
        affine_subspace_sup(left_span, right_span)
    affine_span_list_add_eq_sup(a, right, left)
    affine_span(a, affine_span_list_src[P](right + left)) =
        affine_subspace_sup(right_span, left_span)
    affine_span(a, affine_span_list_src[P](left + right)) =
        affine_span(a, affine_span_list_src[P](right + left))
}

/// A concatenated-list span is contained in an affine subspace exactly when
/// the spans of both parts are contained in it.
theorem affine_span_list_add_subset_iff[V: AddCommGroup, P](
    a: AffineSpace[V, P],
    left: List[P],
    right: List[P],
    t: AffineSubspace[V, P]
) {
    t.space = a implies
        (affine_subspace_subset(
            affine_span(a, affine_span_list_src[P](left + right)), t) =
        (affine_subspace_subset(
            affine_span(a, affine_span_list_src[P](left)), t)
        and affine_subspace_subset(
            affine_span(a, affine_span_list_src[P](right)), t)))
} by {
    if t.space = a {
        let left_span = affine_span(a, affine_span_list_src[P](left))
        let right_span = affine_span(a, affine_span_list_src[P](right))
        let add_span =
            affine_span(a, affine_span_list_src[P](left + right))

        affine_span_space(a, affine_span_list_src[P](left))
        left_span.space = a
        t.space = left_span.space
        affine_subspace_sup_subset_iff(left_span, right_span, t)
        affine_subspace_subset(affine_subspace_sup(left_span, right_span), t) =
            (affine_subspace_subset(left_span, t)
                and affine_subspace_subset(right_span, t))

        affine_span_list_add_eq_sup(a, left, right)
        add_span = affine_subspace_sup(left_span, right_span)
        affine_subspace_subset(add_span, t) =
            (affine_subspace_subset(left_span, t)
                and affine_subspace_subset(right_span, t))
        add_span =
            affine_span(a, affine_span_list_src[P](left + right))
        left_span = affine_span(a, affine_span_list_src[P](left))
        right_span = affine_span(a, affine_span_list_src[P](right))
        affine_subspace_subset(
            affine_span(a, affine_span_list_src[P](left + right)), t) =
            (affine_subspace_subset(
                affine_span(a, affine_span_list_src[P](left)), t)
            and affine_subspace_subset(
                affine_span(a, affine_span_list_src[P](right)), t))
    }
}

/// A concatenated-list span equals the span of its left part exactly when the
/// span of the right part is contained in the left span.
theorem affine_span_list_add_eq_left_iff[V: AddCommGroup, P](
    a: AffineSpace[V, P], left: List[P], right: List[P]
) {
    affine_subspace_subset(
        affine_span(a, affine_span_list_src[P](right)),
        affine_span(a, affine_span_list_src[P](left))) =
    (affine_span(a, affine_span_list_src[P](left + right)) =
        affine_span(a, affine_span_list_src[P](left)))
} by {
    let left_span = affine_span(a, affine_span_list_src[P](left))
    let right_span = affine_span(a, affine_span_list_src[P](right))
    let add_span =
        affine_span(a, affine_span_list_src[P](left + right))

    affine_span_space(a, affine_span_list_src[P](left))
    left_span.space = a
    affine_span_space(a, affine_span_list_src[P](right))
    right_span.space = a
    left_span.space = right_span.space
    affine_subspace_subset_iff_sup_eq_left(left_span, right_span)
    affine_subspace_subset(right_span, left_span) =
        (affine_subspace_sup(left_span, right_span) = left_span)

    affine_span_list_add_eq_sup(a, left, right)
    add_span = affine_subspace_sup(left_span, right_span)
    (add_span = left_span) =
        affine_subspace_subset(right_span, left_span)
    add_span = affine_span(a, affine_span_list_src[P](left + right))
    left_span = affine_span(a, affine_span_list_src[P](left))
    right_span = affine_span(a, affine_span_list_src[P](right))
    affine_subspace_subset(
        affine_span(a, affine_span_list_src[P](right)),
        affine_span(a, affine_span_list_src[P](left))) =
    (affine_span(a, affine_span_list_src[P](left + right)) =
        affine_span(a, affine_span_list_src[P](left)))
}

/// Adjoining points that already belong to a finite affine span does not
/// change that span.
theorem affine_span_list_add_eq_left_of_right_mem_span[V: AddCommGroup, P](
    a: AffineSpace[V, P], left: List[P], right: List[P]
) {
    (forall(x: P) {
        right.contains(x) implies
            affine_span(
                a, affine_span_list_src[P](left)).contains(x)
    }) implies
        affine_span(a, affine_span_list_src[P](left + right)) =
            affine_span(a, affine_span_list_src[P](left))
} by {
    if forall(x: P) {
        right.contains(x) implies
            affine_span(
                a, affine_span_list_src[P](left)).contains(x)
    } {
        let left_span = affine_span(a, affine_span_list_src[P](left))
        let right_span = affine_span(a, affine_span_list_src[P](right))
        affine_span_space(a, affine_span_list_src[P](left))
        left_span.space = a
        affine_span_list_subset_iff(a, right, left_span)
        affine_subspace_subset(right_span, left_span)
        affine_span_list_add_eq_left_iff(a, left, right)
        affine_span(a, affine_span_list_src[P](left + right)) =
            affine_span(a, affine_span_list_src[P](left))
    }
}

/// Repeating a finite list of points does not change its affine span.
theorem affine_span_list_add_self_eq[V: AddCommGroup, P](
    a: AffineSpace[V, P], points: List[P]
) {
    affine_span(a, affine_span_list_src[P](points + points)) =
        affine_span(a, affine_span_list_src[P](points))
} by {
    forall(x: P) {
        if points.contains(x) {
            affine_span_list_src_iff(points, x)
            affine_span_list_src(points, x)
            affine_span_contains_src(
                a, affine_span_list_src[P](points), x)
            affine_span(
                a, affine_span_list_src[P](points)).contains(x)
        }
    }
    affine_span_list_add_eq_left_of_right_mem_span(a, points, points)
}

/// Deleting a middle list whose points already belong to the affine span of
/// the remaining lists does not change the affine span.
theorem affine_span_list_delete_contained_middle[V: AddCommGroup, P](
    a: AffineSpace[V, P],
    left: List[P],
    middle: List[P],
    right: List[P]
) {
    (forall(x: P) {
        middle.contains(x) implies
            affine_span(
                a, affine_span_list_src[P](left + right)).contains(x)
    }) implies
    affine_span(
        a, affine_span_list_src[P]((left + middle) + right)) =
        affine_span(a, affine_span_list_src[P](left + right))
} by {
    if forall(x: P) {
        middle.contains(x) implies
            affine_span(
                a, affine_span_list_src[P](left + right)).contains(x)
    } {
        add_assoc(left, middle, right)
        (left + middle) + right = left + (middle + right)
        add_assoc(left, right, middle)
        (left + right) + middle = left + (right + middle)

        affine_span_list_add_comm(a, middle, right)
        affine_span(a, affine_span_list_src[P](middle + right)) =
            affine_span(a, affine_span_list_src[P](right + middle))
        affine_span_list_add_right_congr(
            a, left, middle + right, right + middle)
        affine_span(
            a, affine_span_list_src[P](left + (middle + right))) =
            affine_span(
                a, affine_span_list_src[P](left + (right + middle)))
        affine_span(
            a, affine_span_list_src[P]((left + middle) + right)) =
            affine_span(
                a, affine_span_list_src[P]((left + right) + middle))

        affine_span_list_add_eq_left_of_right_mem_span(
            a, left + right, middle)
        affine_span(
            a, affine_span_list_src[P]((left + right) + middle)) =
            affine_span(a, affine_span_list_src[P](left + right))
        affine_span(
            a, affine_span_list_src[P]((left + middle) + right)) =
            affine_span(a, affine_span_list_src[P](left + right))
    }
}

/// Replacing a middle list by a list with the same affine span does not
/// change the affine span of the surrounding concatenation.
theorem affine_span_list_middle_congr[V: AddCommGroup, P](
    a: AffineSpace[V, P],
    left: List[P],
    middle: List[P],
    replacement: List[P],
    right: List[P]
) {
    affine_span(a, affine_span_list_src[P](middle)) =
        affine_span(a, affine_span_list_src[P](replacement)) implies
    affine_span(
        a, affine_span_list_src[P]((left + middle) + right)) =
        affine_span(
            a, affine_span_list_src[P]((left + replacement) + right))
} by {
    if affine_span(a, affine_span_list_src[P](middle)) =
        affine_span(a, affine_span_list_src[P](replacement)) {
        add_assoc(left, middle, right)
        (left + middle) + right = left + (middle + right)
        add_assoc(left, replacement, right)
        (left + replacement) + right =
            left + (replacement + right)

        affine_span_list_add_left_congr(
            a, middle, replacement, right)
        affine_span(a, affine_span_list_src[P](middle + right)) =
            affine_span(
                a, affine_span_list_src[P](replacement + right))
        affine_span_list_add_right_congr(
            a, left, middle + right, replacement + right)
        affine_span(
            a, affine_span_list_src[P](left + (middle + right))) =
            affine_span(
                a, affine_span_list_src[P](
                    left + (replacement + right)))
        affine_span(
            a, affine_span_list_src[P]((left + middle) + right)) =
            affine_span(
                a, affine_span_list_src[P](
                    (left + replacement) + right))
    }
}

/// If the left span is contained in the right span, the concatenated-list
/// span equals the right span.
theorem affine_span_list_add_eq_right_of_subset[V: AddCommGroup, P](
    a: AffineSpace[V, P], left: List[P], right: List[P]
) {
    affine_subspace_subset(
        affine_span(a, affine_span_list_src[P](left)),
        affine_span(a, affine_span_list_src[P](right))) implies
    affine_span(a, affine_span_list_src[P](left + right)) =
        affine_span(a, affine_span_list_src[P](right))
} by {
    if affine_subspace_subset(
        affine_span(a, affine_span_list_src[P](left)),
        affine_span(a, affine_span_list_src[P](right))) {
        let left_span = affine_span(a, affine_span_list_src[P](left))
        let right_span = affine_span(a, affine_span_list_src[P](right))
        let add_span =
            affine_span(a, affine_span_list_src[P](left + right))

        affine_span_space(a, affine_span_list_src[P](left))
        left_span.space = a
        affine_span_space(a, affine_span_list_src[P](right))
        right_span.space = a
        left_span.space = right_span.space
        affine_subspace_subset(left_span, right_span)
        affine_subspace_sup_eq_right_of_subset(left_span, right_span)
        affine_subspace_sup(left_span, right_span) = right_span

        affine_span_list_add_eq_sup(a, left, right)
        add_span = affine_subspace_sup(left_span, right_span)
        add_span = right_span
        affine_span(a, affine_span_list_src[P](left + right)) =
            affine_span(a, affine_span_list_src[P](right))
    }
}

/// If the concatenated-list span equals the right span, the left span is
/// contained in the right span.
theorem affine_span_list_left_subset_of_add_eq_right[V: AddCommGroup, P](
    a: AffineSpace[V, P], left: List[P], right: List[P]
) {
    affine_span(a, affine_span_list_src[P](left + right)) =
        affine_span(a, affine_span_list_src[P](right)) implies
    affine_subspace_subset(
        affine_span(a, affine_span_list_src[P](left)),
        affine_span(a, affine_span_list_src[P](right)))
} by {
    if affine_span(a, affine_span_list_src[P](left + right)) =
        affine_span(a, affine_span_list_src[P](right)) {
        let left_span = affine_span(a, affine_span_list_src[P](left))
        let right_span = affine_span(a, affine_span_list_src[P](right))
        let add_span =
            affine_span(a, affine_span_list_src[P](left + right))

        affine_span_list_add_eq_sup(a, left, right)
        add_span = affine_subspace_sup(left_span, right_span)
        affine_subspace_subset_sup_left(left_span, right_span)
        affine_subspace_subset(left_span, add_span)
        add_span = right_span
        affine_subspace_subset(left_span, right_span)
        affine_subspace_subset(
            affine_span(a, affine_span_list_src[P](left)),
            affine_span(a, affine_span_list_src[P](right)))
    }
}

/// The affine span of the full source set equals the full affine subspace.
theorem affine_span_univ_eq[V: AddCommGroup, P](a: AffineSpace[V, P]) {
    affine_span(a, affine_subspace_univ_contains[P]) = affine_subspace_univ(a)
} by {
    let lhs = affine_span(a, affine_subspace_univ_contains[P])
    let rhs = affine_subspace_univ(a)
    affine_span_space(a, affine_subspace_univ_contains[P])
    lhs.space = a
    forall(x: P) {
        affine_span_contains_src(a, affine_subspace_univ_contains[P], x)
        affine_subspace_univ_contains_all(a, x)
        lhs.contains(x) = rhs.contains(x)
    }
    rhs.space = a
    affine_subspace_ext(lhs, rhs)
}
