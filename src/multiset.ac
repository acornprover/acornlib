from list import List
from nat import Nat, add_to_zero, alt_suc_ne_zero
numerals Nat

/// A multiset (bag) that can contain multiple copies of the same element.
/// Each element has a multiplicity indicating how many times it appears.
structure Multiset[T] {
    /// A function taking each element to a count of how many times it appears in the multiset.
    multiplicity: T -> Nat
}

/// The everywhere-zero multiplicity function.
define empty_count[T](x: T) -> Nat {
    Nat.0
}

/// The multiplicity function with one copy at one chosen element.
define singleton_count[T](item: T, x: T) -> Nat {
    if x = item {
        Nat.1
    } else {
        Nat.0
    }
}

/// The multiplicity function obtained by adding one copy of an element.
define inserted_count[T](a: Multiset[T], item: T, x: T) -> Nat {
    if x = item {
        a.multiplicity(x).suc
    } else {
        a.multiplicity(x)
    }
}

/// The pointwise sum of two multiset multiplicity functions.
define sum_count[T](a: Multiset[T], b: Multiset[T], x: T) -> Nat {
    a.multiplicity(x) + b.multiplicity(x)
}

/// The multiplicity function obtained by keeping only elements satisfying a predicate.
define filtered_count[T](a: Multiset[T], pred: T -> Bool, x: T) -> Nat {
    if pred(x) {
        a.multiplicity(x)
    } else {
        Nat.0
    }
}

/// Multiset extensionality: two multisets are equal when every element has the same multiplicity.
theorem multiset_ext[T](a: Multiset[T], b: Multiset[T]) {
    (forall(x: T) { a.multiplicity(x) = b.multiplicity(x) }) implies a = b
} by {
    if forall(x: T) { a.multiplicity(x) = b.multiplicity(x) } {
        a.multiplicity = b.multiplicity
    }
}

/// Equality of multisets is exactly pointwise equality of multiplicities.
theorem multiset_eq_iff_multiplicity_eq[T](a: Multiset[T], b: Multiset[T]) {
    (a = b) = forall(x: T) { a.multiplicity(x) = b.multiplicity(x) }
} by {
    if a = b {
        forall(x: T) {
            a.multiplicity(x) = b.multiplicity(x)
        }
    }
    if forall(x: T) { a.multiplicity(x) = b.multiplicity(x) } {
        multiset_ext(a, b)
        a = b
    }
}

attributes Multiset[T] {
    /// Multiset extensionality from pointwise equality of multiplicity.
    let ext = multiset_ext[T]

    /// The empty multiset, whose multiplicity is zero at every element.
    let empty: Multiset[T] = Multiset.new(empty_count[T])

    /// The multiset containing exactly one copy of the given element.
    let singleton: T -> Multiset[T] = function(item: T) {
        Multiset.new(singleton_count(item))
    }

    /// The multiset whose multiplicities are the counts of elements in a list.
    let from_list: List[T] -> Multiset[T] = function(items: List[T]) {
        Multiset.new(function(x: T) {
            items.count(x)
        })
    }

    /// True if the multiset contains at least one copy of the given item.
    define contains(self, item: T) -> Bool {
        self.multiplicity(item) != 0
    }

    /// Adds one more copy of the given item to the multiset.
    define insert(self, item: T) -> Multiset[T] {
        Multiset.new(inserted_count(self, item))
    }

    /// The pointwise sum of multiplicities, also called bag union.
    define add(self, other: Multiset[T]) -> Multiset[T] {
        Multiset.new(sum_count(self, other))
    }

    /// True if the multiset is empty (contains no elements).
    define is_empty(self) -> Bool {
        forall(x: T) {
            self.multiplicity(x) = 0
        }
    }

    /// True if any element appears more than once in the multiset.
    define has_duplicates(self) -> Bool {
        exists(x: T) {
            self.multiplicity(x) > 1
        }
    }

    /// Keep only the copies of elements satisfying a predicate.
    define filter(self, pred: T -> Bool) -> Multiset[T] {
        Multiset.new(filtered_count(self, pred))
    }
}

/// The empty multiset has zero multiplicity at every element.
theorem empty_multiplicity[T](item: T) {
    Multiset.empty[T].multiplicity(item) = Nat.0
} by {
}

/// A singleton has multiplicity one at its chosen element.
theorem singleton_multiplicity_self[T](item: T) {
    Multiset.singleton(item).multiplicity(item) = Nat.1
} by {
    singleton_count(item, item) = Nat.1
}

/// A singleton has multiplicity zero away from its chosen element.
theorem singleton_multiplicity_other[T](item: T, other: T) {
    item != other implies Multiset.singleton(item).multiplicity(other) = Nat.0
} by {
    if item != other {
        Multiset.singleton(item).multiplicity(other) = Nat.0
    }
}

/// Inserting an element increases its own multiplicity by one.
theorem insert_multiplicity_self[T](a: Multiset[T], item: T) {
    a.insert(item).multiplicity(item) = a.multiplicity(item).suc
} by {
}

/// Inserting one element leaves all other multiplicities unchanged.
theorem insert_multiplicity_other[T](a: Multiset[T], item: T, other: T) {
    item != other implies a.insert(item).multiplicity(other) = a.multiplicity(other)
} by {
    if item != other {
        inserted_count(a, item, other) = a.multiplicity(other)
        a.insert(item).multiplicity(other) = a.multiplicity(other)
    }
}

/// The pointwise sum has multiplicity equal to the sum of multiplicities.
theorem add_multiplicity[T](a: Multiset[T], b: Multiset[T], item: T) {
    (a + b).multiplicity(item) = a.multiplicity(item) + b.multiplicity(item)
} by {
}

/// Filtering keeps the multiplicity of an element satisfying the predicate.
theorem filter_multiplicity_true[T](a: Multiset[T], pred: T -> Bool, item: T) {
    pred(item) implies a.filter(pred).multiplicity(item) = a.multiplicity(item)
} by {
    if pred(item) {
        filtered_count(a, pred, item) = a.multiplicity(item)
        a.filter(pred).multiplicity(item) = a.multiplicity(item)
    }
}

/// Filtering gives multiplicity zero to an element not satisfying the predicate.
theorem filter_multiplicity_false[T](a: Multiset[T], pred: T -> Bool, item: T) {
    not pred(item) implies a.filter(pred).multiplicity(item) = Nat.0
} by {
    if not pred(item) {
        filtered_count(a, pred, item) = Nat.0
        a.filter(pred).multiplicity(item) = Nat.0
    }
}

/// Containment in a filtered multiset is containment together with the predicate.
theorem filter_contains_iff[T](a: Multiset[T], pred: T -> Bool, item: T) {
    a.filter(pred).contains(item) = (a.contains(item) and pred(item))
} by {
    if pred(item) {
        filter_multiplicity_true(a, pred, item)
        if a.contains(item) {
            a.multiplicity(item) != Nat.0
            a.filter(pred).multiplicity(item) != Nat.0
            a.contains(item) and pred(item)
        }
        if a.filter(pred).contains(item) {
            a.filter(pred).multiplicity(item) != Nat.0
            a.contains(item) and pred(item)
        }
    }
    if not pred(item) {
        filter_multiplicity_false(a, pred, item)
        not (a.contains(item) and pred(item))
    }
    if a.filter(pred).contains(item) {
        if not pred(item) {
            filter_multiplicity_false(a, pred, item)
            false
        }
        filter_multiplicity_true(a, pred, item)
        a.contains(item) and pred(item)
    }
    if a.contains(item) and pred(item) {
        filter_multiplicity_true(a, pred, item)
        a.filter(pred).contains(item)
    }
}

/// Filtering commutes with pointwise multiset addition.
theorem filter_add[T](a: Multiset[T], b: Multiset[T], pred: T -> Bool) {
    (a + b).filter(pred) = a.filter(pred) + b.filter(pred)
} by {
    forall(x: T) {
        if pred(x) {
            filter_multiplicity_true(a + b, pred, x)
            add_multiplicity(a, b, x)
            filter_multiplicity_true(a, pred, x)
            a.filter(pred).multiplicity(x) = a.multiplicity(x)
            filter_multiplicity_true(b, pred, x)
            b.filter(pred).multiplicity(x) = b.multiplicity(x)
            add_multiplicity(a.filter(pred), b.filter(pred), x)
            (a + b).filter(pred).multiplicity(x) =
                (a.filter(pred) + b.filter(pred)).multiplicity(x)
        }
        if not pred(x) {
            filter_multiplicity_false(a + b, pred, x)
            filter_multiplicity_false(a, pred, x)
            filter_multiplicity_false(b, pred, x)
            add_multiplicity(a.filter(pred), b.filter(pred), x)
            (a + b).filter(pred).multiplicity(x) =
                (a.filter(pred) + b.filter(pred)).multiplicity(x)
        }
        (a + b).filter(pred).multiplicity(x) =
            (a.filter(pred) + b.filter(pred)).multiplicity(x)
    }
    forall(x: T) {
        (a + b).filter(pred).multiplicity(x) =
            (a.filter(pred) + b.filter(pred)).multiplicity(x)
    }
    multiset_ext((a + b).filter(pred), a.filter(pred) + b.filter(pred))
}

/// Adding the empty multiset on the right changes nothing.
theorem multiset_add_empty_right[T](a: Multiset[T]) {
    a + Multiset.empty[T] = a
} by {
    forall(x: T) {
        Multiset.empty[T].multiplicity(x) = Nat.0
        (a + Multiset.empty[T]).multiplicity(x) = a.multiplicity(x)
    }
    multiset_ext(a + Multiset.empty[T], a)
}

/// Adding the empty multiset on the left changes nothing.
theorem multiset_empty_add[T](a: Multiset[T]) {
    Multiset.empty[T] + a = a
} by {
    forall(x: T) {
        Multiset.empty[T].multiplicity(x) = Nat.0
        Nat.0 + a.multiplicity(x) = a.multiplicity(x)
        (Multiset.empty[T] + a).multiplicity(x) = a.multiplicity(x)
    }
    multiset_ext(Multiset.empty[T] + a, a)
}

/// Pointwise addition of multisets is commutative.
theorem multiset_add_comm[T](a: Multiset[T], b: Multiset[T]) {
    a + b = b + a
} by {
    forall(x: T) {
        a.multiplicity(x) + b.multiplicity(x) = b.multiplicity(x) + a.multiplicity(x)
        (a + b).multiplicity(x) = (b + a).multiplicity(x)
    }
    multiset_ext(a + b, b + a)
}

/// Pointwise addition of multisets is associative.
theorem multiset_add_assoc[T](a: Multiset[T], b: Multiset[T], c: Multiset[T]) {
    (a + b) + c = a + (b + c)
} by {
    forall(x: T) {
        (a.multiplicity(x) + b.multiplicity(x)) + c.multiplicity(x) =
            a.multiplicity(x) + (b.multiplicity(x) + c.multiplicity(x))
        ((a + b) + c).multiplicity(x) = (a + (b + c)).multiplicity(x)
    }
    multiset_ext((a + b) + c, a + (b + c))
}

/// Containment is equivalent to having positive multiplicity.
theorem contains_iff_positive_multiplicity[T](a: Multiset[T], item: T) {
    a.contains(item) = (a.multiplicity(item) > Nat.0)
} by {
    if a.contains(item) {
        if not a.multiplicity(item) > Nat.0 {
            a.multiplicity(item) = Nat.0
            false
        }
    }
    if a.multiplicity(item) > Nat.0 {
        if not a.contains(item) {
            false
        }
    }
}

/// The empty multiset contains no elements.
theorem empty_not_contains[T](item: T) {
    not Multiset.empty[T].contains(item)
} by {
    empty_multiplicity[T](item)
}

/// A singleton contains exactly its chosen element.
theorem singleton_contains_iff[T](item: T, x: T) {
    Multiset.singleton(item).contains(x) = (x = item)
} by {
    if x = item {
        singleton_multiplicity_self(item)
        alt_suc_ne_zero(Nat.0)
        Multiset.singleton(item).multiplicity(x) != Nat.0
        Multiset.singleton(item).contains(x)
    }
    if x != item {
        singleton_multiplicity_other(item, x)
        not x = item
    }
}

/// Inserting an element makes the resulting multiset contain that element.
theorem insert_contains_self[T](a: Multiset[T], item: T) {
    a.insert(item).contains(item)
} by {
    insert_multiplicity_self(a, item)
    alt_suc_ne_zero(a.multiplicity(item))
}

/// Inserting an element makes containment equivalent to being the inserted element or already contained.
theorem insert_contains_iff[T](a: Multiset[T], item: T, x: T) {
    a.insert(item).contains(x) = (x = item or a.contains(x))
} by {
    if x = item {
        insert_contains_self(a, item)
        x = item or a.contains(x)
    }
    if x != item {
        item != x
        insert_multiplicity_other(a, item, x)
        if a.contains(x) {
            x = item or a.contains(x)
        }
        if a.insert(item).contains(x) {
            x = item or a.contains(x)
        }
        if not a.contains(x) {
            not (x = item or a.contains(x))
        }
        a.insert(item).contains(x) = (x = item or a.contains(x))
    }
    x = item or x != item
}

/// If the left summand contains an element, then the sum contains it.
theorem add_contains_left[T](a: Multiset[T], b: Multiset[T], x: T) {
    a.contains(x) implies (a + b).contains(x)
} by {
    if a.contains(x) {
        add_multiplicity(a, b, x)
        if (a + b).multiplicity(x) = Nat.0 {
            add_to_zero(a.multiplicity(x), b.multiplicity(x))
            a.multiplicity(x) = Nat.0
            false
        }
        (a + b).contains(x)
    }
}

/// If the right summand contains an element, then the sum contains it.
theorem add_contains_right[T](a: Multiset[T], b: Multiset[T], x: T) {
    b.contains(x) implies (a + b).contains(x)
} by {
    if b.contains(x) {
        add_multiplicity(a, b, x)
        if (a + b).multiplicity(x) = Nat.0 {
            add_to_zero(a.multiplicity(x), b.multiplicity(x))
            b.multiplicity(x) = Nat.0
            false
        }
        (a + b).contains(x)
    }
}

/// Containment in a sum is containment in either summand.
theorem add_contains_iff[T](a: Multiset[T], b: Multiset[T], x: T) {
    (a + b).contains(x) = (a.contains(x) or b.contains(x))
} by {
    if a.contains(x) or b.contains(x) {
        if a.contains(x) {
            add_contains_left(a, b, x)
            (a + b).contains(x)
        }
        if b.contains(x) {
            add_contains_right(a, b, x)
            (a + b).contains(x)
        }
        (a + b).contains(x)
    }
    if not (a.contains(x) or b.contains(x)) {
        a.multiplicity(x) = Nat.0
        b.multiplicity(x) = Nat.0
        add_multiplicity(a, b, x)
        not (a + b).contains(x)
    }
}

/// A sum is empty exactly when both summands are empty.
theorem multiset_add_is_empty_iff[T](a: Multiset[T], b: Multiset[T]) {
    (a + b).is_empty = (a.is_empty and b.is_empty)
} by {
    if (a + b).is_empty {
        forall(x: T) {
            (a + b).multiplicity(x) = Nat.0
            add_multiplicity(a, b, x)
            add_to_zero(a.multiplicity(x), b.multiplicity(x))
            a.multiplicity(x) = Nat.0
        }
        a.is_empty
        forall(x: T) {
            (a + b).multiplicity(x) = Nat.0
            add_multiplicity(a, b, x)
            add_to_zero(a.multiplicity(x), b.multiplicity(x))
            b.multiplicity(x) = Nat.0
        }
        b.is_empty
        a.is_empty and b.is_empty
    }
    if a.is_empty and b.is_empty {
        forall(x: T) {
            a.multiplicity(x) = Nat.0
            b.multiplicity(x) = Nat.0
            add_multiplicity(a, b, x)
            (a + b).multiplicity(x) = Nat.0
        }
        (a + b).is_empty
    }
}

/// A singleton multiset is not empty.
theorem singleton_not_empty[T](item: T) {
    not Multiset.singleton(item).is_empty
} by {
    if Multiset.singleton(item).is_empty {
        Multiset.singleton(item).multiplicity(item) = Nat.0
        singleton_multiplicity_self(item)
        alt_suc_ne_zero(Nat.0)
        false
    }
}

/// Inserting an element always yields a nonempty multiset.
theorem insert_not_empty[T](a: Multiset[T], item: T) {
    not a.insert(item).is_empty
} by {
    if a.insert(item).is_empty {
        a.insert(item).multiplicity(item) = Nat.0
        insert_multiplicity_self(a, item)
        alt_suc_ne_zero(a.multiplicity(item))
        false
    }
}

/// A multiset is empty exactly when it equals the empty multiset.
theorem is_empty_iff_eq_empty[T](a: Multiset[T]) {
    a.is_empty = (a = Multiset.empty[T])
} by {
    if a.is_empty {
        forall(x: T) {
            a.multiplicity(x) = Nat.0
            a.multiplicity(x) = Multiset.empty[T].multiplicity(x)
        }
        multiset_ext(a, Multiset.empty[T])
        a = Multiset.empty[T]
    }
    if a = Multiset.empty[T] {
        forall(x: T) {
            a.multiplicity(x) = Nat.0
        }
        a.is_empty
    }
}

/// The multiplicity in `from_list` is the list count.
theorem from_list_multiplicity[T](items: List[T], item: T) {
    Multiset.from_list(items).multiplicity(item) = items.count(item)
}

/// The empty list represents the empty multiset.
theorem from_list_nil[T] {
    Multiset.from_list(List.nil[T]) = Multiset.empty[T]
} by {
    forall(x: T) {
        List.nil[T].count(x) = Nat.0
        Multiset.from_list(List.nil[T]).multiplicity(x) = Multiset.empty[T].multiplicity(x)
    }
    multiset_ext(Multiset.from_list(List.nil[T]), Multiset.empty[T])
}
