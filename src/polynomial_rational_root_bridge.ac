from nat import Nat
from int import Int
from rat import Rat, add_int_eq_int_add, mul_int_eq_int_mul, from_int_zero, from_int_cancel
from algebra.field.field import mul_inverse_left
from data.basic.functions import function_extensionality
from algebra.comm_monoid_rearrange import mul_swap_inner
from polynomial import coeff_tail, coeff_eval
from polynomial_rational_root import coeff_hom_eval, coeff_hom_eval_suc, coeff_hom_eval_zero,
    hom_root_numerator_divides, hom_root_denominator_divides
from data.int.int_coprime import is_coprime

numerals Rat

/// The rational coefficient function of an integer one.
define rat_coeffs(c: Nat -> Int) -> (Nat -> Rat) {
    function(i: Nat) {
        Rat.from_int(c(i))
    }
}

/// A rational coefficient is the embedding of the integer one.
theorem rat_coeffs_apply(c: Nat -> Int, i: Nat) {
    rat_coeffs(c)(i) = Rat.from_int(c(i))
}

/// Dropping the constant coefficient commutes with the embedding.
theorem rat_coeffs_tail(c: Nat -> Int) {
    coeff_tail(rat_coeffs(c)) = rat_coeffs(coeff_tail(c))
} by {
    forall(i: Nat) {
        (coeff_tail(rat_coeffs(c))(i) = rat_coeffs(c)(i.suc))
        rat_coeffs_apply(c, i.suc)
        (rat_coeffs(c)(i.suc) = Rat.from_int(c(i.suc)))
        (coeff_tail(c)(i) = c(i.suc))
        rat_coeffs_apply(coeff_tail(c), i)
        (rat_coeffs(coeff_tail(c))(i) = Rat.from_int(coeff_tail(c)(i)))
        coeff_tail(rat_coeffs(c))(i) = rat_coeffs(coeff_tail(c))(i)
    }
    function_extensionality[Nat, Rat](coeff_tail(rat_coeffs(c)), rat_coeffs(coeff_tail(c)))
    coeff_tail(rat_coeffs(c)) = rat_coeffs(coeff_tail(c))
}

/// The embedding of the integers into the rationals preserves powers.
theorem rat_from_int_pow(a: Int, n: Nat) {
    Rat.from_int(a).pow(n) = Rat.from_int(a.pow(n))
} by {
    define w(k: Nat) -> Bool {
        Rat.from_int(a).pow(k) = Rat.from_int(a.pow(k))
    }
    (Rat.from_int(a).pow(Nat.0) = Rat.1)
    (a.pow(Nat.0) = Int.1)
    (Rat.from_int(Int.1) = Rat.1)
    w(Nat.0)
    forall(k: Nat) {
        if w(k) {
            (Rat.from_int(a).pow(k) = Rat.from_int(a.pow(k)))
            (Rat.from_int(a).pow(k.suc) = Rat.from_int(a).pow(k) * Rat.from_int(a))
            mul_int_eq_int_mul(a.pow(k), a)
            (Rat.from_int(a.pow(k)) * Rat.from_int(a) = Rat.from_int(a.pow(k) * a))
            (a.pow(k.suc) = a.pow(k) * a)
            Rat.from_int(a).pow(k.suc) = Rat.from_int(a.pow(k.suc))
            w(k.suc)
        }
        (w(k) implies w(k.suc))
    }
    w(Nat.0) and forall(k: Nat) {
        w(k) implies w(k.suc)
    }
    Nat.induction(w)
    w(n)
    Rat.from_int(a).pow(n) = Rat.from_int(a.pow(n))
}

/// A nonzero integer embeds as a nonzero rational.
theorem rat_from_int_ne_zero(b: Int) {
    b != Int.0 implies Rat.from_int(b) != Rat.0
} by {
    if b != Int.0 {
        if Rat.from_int(b) = Rat.0 {
            from_int_zero
            (Rat.from_int(Int.0) = Rat.0)
            Rat.from_int(b) = Rat.from_int(Int.0)
            from_int_cancel(b, Int.0)
            b = Int.0
            false
        }
        Rat.from_int(b) != Rat.0
    }
}

/// The fraction times its denominator is its numerator.
theorem rat_fraction_times_denom(a: Int, b: Int) {
    b != Int.0 implies (Rat.from_int(a) / Rat.from_int(b)) * Rat.from_int(b)
        = Rat.from_int(a)
} by {
    if b != Int.0 {
        rat_from_int_ne_zero(b)
        Rat.from_int(b) != Rat.0
        ((Rat.from_int(a) / Rat.from_int(b))
            = Rat.from_int(a) * Rat.from_int(b).inverse)
        ((Rat.from_int(a) * Rat.from_int(b).inverse) * Rat.from_int(b)
            = Rat.from_int(a) * (Rat.from_int(b).inverse * Rat.from_int(b)))
        mul_inverse_left[Rat](Rat.from_int(b))
        (Rat.from_int(b).inverse * Rat.from_int(b) = Rat.1)
        (Rat.from_int(a) * Rat.1 = Rat.from_int(a))
        (Rat.from_int(a) / Rat.from_int(b)) * Rat.from_int(b) = Rat.from_int(a)
    }
}

/// The rational value at a fraction, cleared of denominators, is the homogeneous value.
///
/// This is what makes `coeff_hom_eval` the cleared equation rather than merely a sum that looks
/// like one. Induction along the coefficient list, generalised over the coefficient function
/// since the step applies the hypothesis at the tail.
theorem coeff_eval_fraction_cleared(c: Nat -> Int, a: Int, b: Int, n: Nat) {
    b != Int.0 implies
        coeff_eval(rat_coeffs(c), Rat.from_int(a) / Rat.from_int(b), n.suc)
            * Rat.from_int(b).pow(n)
        = Rat.from_int(coeff_hom_eval(c, a, b, n.suc))
} by {
    if b != Int.0 {
        rat_fraction_times_denom(a, b)
        ((Rat.from_int(a) / Rat.from_int(b)) * Rat.from_int(b) = Rat.from_int(a))
        define w(k: Nat) -> Bool {
            forall(d: Nat -> Int) {
                coeff_eval(rat_coeffs(d), Rat.from_int(a) / Rat.from_int(b), k.suc)
                    * Rat.from_int(b).pow(k)
                = Rat.from_int(coeff_hom_eval(d, a, b, k.suc))
            }
        }
        forall(d: Nat -> Int) {
            (coeff_eval(rat_coeffs(d), Rat.from_int(a) / Rat.from_int(b), Nat.zero.suc)
                = rat_coeffs(d)(Nat.0) + (Rat.from_int(a) / Rat.from_int(b))
                    * coeff_eval(coeff_tail(rat_coeffs(d)),
                        Rat.from_int(a) / Rat.from_int(b), Nat.zero))
            (coeff_eval(coeff_tail(rat_coeffs(d)),
                Rat.from_int(a) / Rat.from_int(b), Nat.zero) = Rat.0)
            ((Rat.from_int(a) / Rat.from_int(b)) * Rat.0 = Rat.0)
            rat_coeffs_apply(d, Nat.0)
            (rat_coeffs(d)(Nat.0) = Rat.from_int(d(Nat.0)))
            (Rat.from_int(d(Nat.0)) + Rat.0 = Rat.from_int(d(Nat.0)))
            (Rat.from_int(b).pow(Nat.0) = Rat.1)
            (Rat.from_int(d(Nat.0)) * Rat.1 = Rat.from_int(d(Nat.0)))
            coeff_hom_eval_suc(d, a, b, Nat.0)
            coeff_hom_eval_zero(coeff_tail(d), a, b)
            (coeff_hom_eval(coeff_tail(d), a, b, Nat.0) = Int.0)
            (b.pow(Nat.0) = Int.1)
            (d(Nat.0) * Int.1 = d(Nat.0))
            (a * Int.0 = Int.0)
            (d(Nat.0) + Int.0 = d(Nat.0))
            (coeff_hom_eval(d, a, b, Nat.0.suc) = d(Nat.0))
            (Nat.zero = Nat.0)
            (coeff_eval(rat_coeffs(d), Rat.from_int(a) / Rat.from_int(b), Nat.0.suc)
                * Rat.from_int(b).pow(Nat.0)
                = Rat.from_int(coeff_hom_eval(d, a, b, Nat.0.suc)))
        }
        w(Nat.0)
        forall(k: Nat) {
            if w(k) {
                (forall(e: Nat -> Int) {
                    coeff_eval(rat_coeffs(e), Rat.from_int(a) / Rat.from_int(b), k.suc)
                        * Rat.from_int(b).pow(k)
                    = Rat.from_int(coeff_hom_eval(e, a, b, k.suc))
                })
                forall(d: Nat -> Int) {
                    (coeff_eval(rat_coeffs(coeff_tail(d)),
                        Rat.from_int(a) / Rat.from_int(b), k.suc)
                        * Rat.from_int(b).pow(k)
                        = Rat.from_int(coeff_hom_eval(coeff_tail(d), a, b, k.suc)))
                    rat_coeffs_tail(d)
                    (coeff_tail(rat_coeffs(d)) = rat_coeffs(coeff_tail(d)))
                    (coeff_eval(rat_coeffs(d), Rat.from_int(a) / Rat.from_int(b), k.suc.suc)
                        = rat_coeffs(d)(Nat.0) + (Rat.from_int(a) / Rat.from_int(b))
                            * coeff_eval(coeff_tail(rat_coeffs(d)),
                                Rat.from_int(a) / Rat.from_int(b), k.suc))
                    (coeff_eval(rat_coeffs(d), Rat.from_int(a) / Rat.from_int(b), k.suc.suc)
                        = rat_coeffs(d)(Nat.0) + (Rat.from_int(a) / Rat.from_int(b))
                            * coeff_eval(rat_coeffs(coeff_tail(d)),
                                Rat.from_int(a) / Rat.from_int(b), k.suc))
                    (Rat.from_int(b).pow(k.suc) = Rat.from_int(b).pow(k) * Rat.from_int(b))
                    coeff_hom_eval_suc(d, a, b, k.suc)
                    (coeff_hom_eval(d, a, b, k.suc.suc)
                        = d(Nat.0) * b.pow(k.suc)
                            + a * coeff_hom_eval(coeff_tail(d), a, b, k.suc))
                    ((rat_coeffs(d)(Nat.0) + (Rat.from_int(a) / Rat.from_int(b))
                        * coeff_eval(rat_coeffs(coeff_tail(d)),
                            Rat.from_int(a) / Rat.from_int(b), k.suc))
                        * (Rat.from_int(b).pow(k) * Rat.from_int(b))
                        = rat_coeffs(d)(Nat.0)
                            * (Rat.from_int(b).pow(k) * Rat.from_int(b))
                        + ((Rat.from_int(a) / Rat.from_int(b))
                            * coeff_eval(rat_coeffs(coeff_tail(d)),
                                Rat.from_int(a) / Rat.from_int(b), k.suc))
                            * (Rat.from_int(b).pow(k) * Rat.from_int(b)))
                    mul_swap_inner[Rat](Rat.from_int(a) / Rat.from_int(b),
                        coeff_eval(rat_coeffs(coeff_tail(d)),
                            Rat.from_int(a) / Rat.from_int(b), k.suc),
                        Rat.from_int(b).pow(k), Rat.from_int(b))
                    (((Rat.from_int(a) / Rat.from_int(b))
                        * coeff_eval(rat_coeffs(coeff_tail(d)),
                            Rat.from_int(a) / Rat.from_int(b), k.suc))
                        * (Rat.from_int(b).pow(k) * Rat.from_int(b))
                        = ((Rat.from_int(a) / Rat.from_int(b)) * Rat.from_int(b).pow(k))
                            * (coeff_eval(rat_coeffs(coeff_tail(d)),
                                Rat.from_int(a) / Rat.from_int(b), k.suc)
                                * Rat.from_int(b)))
                    mul_swap_inner[Rat](Rat.from_int(a) / Rat.from_int(b),
                        coeff_eval(rat_coeffs(coeff_tail(d)),
                            Rat.from_int(a) / Rat.from_int(b), k.suc),
                        Rat.from_int(b), Rat.from_int(b).pow(k))
                    (((Rat.from_int(a) / Rat.from_int(b))
                        * coeff_eval(rat_coeffs(coeff_tail(d)),
                            Rat.from_int(a) / Rat.from_int(b), k.suc))
                        * (Rat.from_int(b) * Rat.from_int(b).pow(k))
                        = ((Rat.from_int(a) / Rat.from_int(b)) * Rat.from_int(b))
                            * (coeff_eval(rat_coeffs(coeff_tail(d)),
                                Rat.from_int(a) / Rat.from_int(b), k.suc)
                                * Rat.from_int(b).pow(k)))
                    (Rat.from_int(b).pow(k) * Rat.from_int(b)
                        = Rat.from_int(b) * Rat.from_int(b).pow(k))
                    (((Rat.from_int(a) / Rat.from_int(b))
                        * coeff_eval(rat_coeffs(coeff_tail(d)),
                            Rat.from_int(a) / Rat.from_int(b), k.suc))
                        * (Rat.from_int(b).pow(k) * Rat.from_int(b))
                        = Rat.from_int(a)
                            * Rat.from_int(coeff_hom_eval(coeff_tail(d), a, b, k.suc)))
                    rat_from_int_pow(b, k.suc)
                    (Rat.from_int(b).pow(k.suc) = Rat.from_int(b.pow(k.suc)))
                    rat_coeffs_apply(d, Nat.0)
                    (rat_coeffs(d)(Nat.0) = Rat.from_int(d(Nat.0)))
                    mul_int_eq_int_mul(d(Nat.0), b.pow(k.suc))
                    (Rat.from_int(d(Nat.0)) * Rat.from_int(b.pow(k.suc))
                        = Rat.from_int(d(Nat.0) * b.pow(k.suc)))
                    mul_int_eq_int_mul(a, coeff_hom_eval(coeff_tail(d), a, b, k.suc))
                    (Rat.from_int(a) * Rat.from_int(coeff_hom_eval(coeff_tail(d), a, b, k.suc))
                        = Rat.from_int(a * coeff_hom_eval(coeff_tail(d), a, b, k.suc)))
                    add_int_eq_int_add(d(Nat.0) * b.pow(k.suc),
                        a * coeff_hom_eval(coeff_tail(d), a, b, k.suc))
                    (Rat.from_int(d(Nat.0) * b.pow(k.suc))
                        + Rat.from_int(a * coeff_hom_eval(coeff_tail(d), a, b, k.suc))
                        = Rat.from_int(d(Nat.0) * b.pow(k.suc)
                            + a * coeff_hom_eval(coeff_tail(d), a, b, k.suc)))
                    (coeff_eval(rat_coeffs(d), Rat.from_int(a) / Rat.from_int(b), k.suc.suc)
                        * Rat.from_int(b).pow(k.suc)
                        = Rat.from_int(coeff_hom_eval(d, a, b, k.suc.suc)))
                }
                w(k.suc)
            }
            (w(k) implies w(k.suc))
        }
        w(Nat.0) and forall(k: Nat) {
            w(k) implies w(k.suc)
        }
        Nat.induction(w)
        w(n)
        (forall(d: Nat -> Int) {
            coeff_eval(rat_coeffs(d), Rat.from_int(a) / Rat.from_int(b), n.suc)
                * Rat.from_int(b).pow(n)
            = Rat.from_int(coeff_hom_eval(d, a, b, n.suc))
        })
        (coeff_eval(rat_coeffs(c), Rat.from_int(a) / Rat.from_int(b), n.suc)
            * Rat.from_int(b).pow(n)
            = Rat.from_int(coeff_hom_eval(c, a, b, n.suc)))
    }
}

/// A product of nonzero rationals is nonzero.
///
/// The rationals are a field, so multiplying by the inverse of one factor recovers the other.
theorem rat_mul_ne_zero(x: Rat, y: Rat) {
    x != Rat.0 and y != Rat.0 implies x * y != Rat.0
} by {
    if x != Rat.0 and y != Rat.0 {
        if x * y = Rat.0 {
            mul_inverse_left[Rat](x)
            (x.inverse * x = Rat.1)
            (x.inverse * (x * y) = (x.inverse * x) * y)
            ((x.inverse * x) * y = y)
            (x.inverse * Rat.0 = Rat.0)
            y = Rat.0
            false
        }
        x * y != Rat.0
    }
}

/// A power of a nonzero rational is nonzero.
theorem rat_pow_ne_zero(x: Rat, n: Nat) {
    x != Rat.0 implies x.pow(n) != Rat.0
} by {
    if x != Rat.0 {
        define w(k: Nat) -> Bool {
            x.pow(k) != Rat.0
        }
        (x.pow(Nat.0) = Rat.1)
        (Rat.1 != Rat.0)
        w(Nat.0)
        forall(k: Nat) {
            if w(k) {
                x.pow(k) != Rat.0
                rat_mul_ne_zero(x.pow(k), x)
                x.pow(k) * x != Rat.0
                (x.pow(k.suc) = x.pow(k) * x)
                x.pow(k.suc) != Rat.0
                w(k.suc)
            }
            (w(k) implies w(k.suc))
        }
        w(Nat.0) and forall(k: Nat) {
            w(k) implies w(k.suc)
        }
        Nat.induction(w)
        w(n)
        x.pow(n) != Rat.0
    }
}

/// A fraction is a root exactly when the cleared value vanishes.
///
/// The bridge between the rational statement and the integer one. A power of a nonzero
/// denominator is nonzero, so clearing it neither creates nor destroys a root.
theorem rat_root_iff_hom_eval_zero(c: Nat -> Int, a: Int, b: Int, n: Nat) {
    b != Int.0 implies
        (coeff_eval(rat_coeffs(c), Rat.from_int(a) / Rat.from_int(b), n.suc) = Rat.0)
            = (coeff_hom_eval(c, a, b, n.suc) = Int.0)
} by {
    if b != Int.0 {
        rat_from_int_ne_zero(b)
        Rat.from_int(b) != Rat.0
        coeff_eval_fraction_cleared(c, a, b, n)
        (coeff_eval(rat_coeffs(c), Rat.from_int(a) / Rat.from_int(b), n.suc)
            * Rat.from_int(b).pow(n)
            = Rat.from_int(coeff_hom_eval(c, a, b, n.suc)))
        rat_from_int_pow(b, n)
        (Rat.from_int(b).pow(n) = Rat.from_int(b.pow(n)))
        rat_pow_ne_zero(Rat.from_int(b), n)
        Rat.from_int(b).pow(n) != Rat.0
        if coeff_eval(rat_coeffs(c), Rat.from_int(a) / Rat.from_int(b), n.suc) = Rat.0 {
            (Rat.0 * Rat.from_int(b).pow(n) = Rat.0)
            Rat.from_int(coeff_hom_eval(c, a, b, n.suc)) = Rat.0
            from_int_zero
            (Rat.from_int(Int.0) = Rat.0)
            Rat.from_int(coeff_hom_eval(c, a, b, n.suc)) = Rat.from_int(Int.0)
            from_int_cancel(coeff_hom_eval(c, a, b, n.suc), Int.0)
            coeff_hom_eval(c, a, b, n.suc) = Int.0
        }
        if coeff_hom_eval(c, a, b, n.suc) = Int.0 {
            from_int_zero
            (Rat.from_int(Int.0) = Rat.0)
            (coeff_eval(rat_coeffs(c), Rat.from_int(a) / Rat.from_int(b), n.suc)
                * Rat.from_int(b).pow(n) = Rat.0)
            if coeff_eval(rat_coeffs(c), Rat.from_int(a) / Rat.from_int(b), n.suc) != Rat.0 {
                rat_mul_ne_zero(
                    coeff_eval(rat_coeffs(c), Rat.from_int(a) / Rat.from_int(b), n.suc),
                    Rat.from_int(b).pow(n))
                (coeff_eval(rat_coeffs(c), Rat.from_int(a) / Rat.from_int(b), n.suc)
                    * Rat.from_int(b).pow(n) != Rat.0)
                false
            }
            coeff_eval(rat_coeffs(c), Rat.from_int(a) / Rat.from_int(b), n.suc) = Rat.0
        }
        ((coeff_eval(rat_coeffs(c), Rat.from_int(a) / Rat.from_int(b), n.suc) = Rat.0)
            = (coeff_hom_eval(c, a, b, n.suc) = Int.0))
    }
}

/// The numerator of a rational root divides the constant coefficient.
///
/// The rational root test, stated at the fraction itself. With the bridge, the integer argument
/// carries over directly.
theorem rat_root_numerator_divides(c: Nat -> Int, a: Int, b: Int, n: Nat) {
    b != Int.0 and is_coprime(a, b)
        and coeff_eval(rat_coeffs(c), Rat.from_int(a) / Rat.from_int(b), n.suc) = Rat.0
        implies a.divides(c(Nat.0))
} by {
    if b != Int.0 and is_coprime(a, b)
        and coeff_eval(rat_coeffs(c), Rat.from_int(a) / Rat.from_int(b), n.suc) = Rat.0 {
        rat_root_iff_hom_eval_zero(c, a, b, n)
        ((coeff_eval(rat_coeffs(c), Rat.from_int(a) / Rat.from_int(b), n.suc) = Rat.0)
            = (coeff_hom_eval(c, a, b, n.suc) = Int.0))
        coeff_hom_eval(c, a, b, n.suc) = Int.0
        hom_root_numerator_divides(c, a, b, n)
        a.divides(c(Nat.0))
    }
}

/// The denominator of a rational root divides the leading coefficient.
theorem rat_root_denominator_divides(c: Nat -> Int, a: Int, b: Int, n: Nat) {
    b != Int.0 and is_coprime(a, b)
        and coeff_eval(rat_coeffs(c), Rat.from_int(a) / Rat.from_int(b), n.suc) = Rat.0
        implies b.divides(c(n))
} by {
    if b != Int.0 and is_coprime(a, b)
        and coeff_eval(rat_coeffs(c), Rat.from_int(a) / Rat.from_int(b), n.suc) = Rat.0 {
        rat_root_iff_hom_eval_zero(c, a, b, n)
        ((coeff_eval(rat_coeffs(c), Rat.from_int(a) / Rat.from_int(b), n.suc) = Rat.0)
            = (coeff_hom_eval(c, a, b, n.suc) = Int.0))
        coeff_hom_eval(c, a, b, n.suc) = Int.0
        hom_root_denominator_divides(c, a, b, n)
        b.divides(c(n))
    }
}

/// A rational root of a monic integer polynomial is an integer.
///
/// The corollary the rational root test is usually used for. The denominator divides the leading
/// coefficient, which is one, so the fraction is already in lowest terms with denominator a
/// unit.
theorem monic_rat_root_denominator_divides_one(c: Nat -> Int, a: Int, b: Int, n: Nat) {
    b != Int.0 and is_coprime(a, b) and c(n) = Int.1
        and coeff_eval(rat_coeffs(c), Rat.from_int(a) / Rat.from_int(b), n.suc) = Rat.0
        implies b.divides(Int.1)
} by {
    if b != Int.0 and is_coprime(a, b) and c(n) = Int.1
        and coeff_eval(rat_coeffs(c), Rat.from_int(a) / Rat.from_int(b), n.suc) = Rat.0 {
        rat_root_denominator_divides(c, a, b, n)
        b.divides(c(n))
        b.divides(Int.1)
    }
}
