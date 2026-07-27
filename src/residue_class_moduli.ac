from int import Int, div_trans, div_from_nat
from list import List
from nat import Nat, divides_trans, divides_self
from pair import Pair
from zmod import int_mod_rel
from number_theory import congruence_class_contains, covers_int, covers_int_cons_left,
    covers_int_cons_right, covers_int_cons_imp, covers_int_nil_false,
    congruence_class_contains_of_congruent

numerals Int

/// True if every modulus of the system divides `m`.
///
/// The condition that makes the whole system periodic with period `m`, which is what lets a
/// covering or disjointness question be settled on a finite stretch of the integers.
define all_moduli_divide(system: List[Pair[Nat, Nat]], m: Nat) -> Bool {
    match system {
        List.nil {
            true
        }
        List.cons(head, tail) {
            head.first.divides(m) and all_moduli_divide(tail, m)
        }
    }
}

/// The head modulus of such a system divides.
theorem all_moduli_divide_head(
    head: Pair[Nat, Nat], tail: List[Pair[Nat, Nat]], m: Nat
) {
    all_moduli_divide(List.cons(head, tail), m) implies head.first.divides(m)
} by {
    if all_moduli_divide(List.cons(head, tail), m) {
        head.first.divides(m)
    }
}

/// The tail of such a system satisfies the condition.
theorem all_moduli_divide_tail(
    head: Pair[Nat, Nat], tail: List[Pair[Nat, Nat]], m: Nat
) {
    all_moduli_divide(List.cons(head, tail), m) implies all_moduli_divide(tail, m)
} by {
    if all_moduli_divide(List.cons(head, tail), m) {
        all_moduli_divide(tail, m)
    }
}

/// The two conditions give the whole system.
theorem all_moduli_divide_cons_intro(
    head: Pair[Nat, Nat], tail: List[Pair[Nat, Nat]], m: Nat
) {
    head.first.divides(m) and all_moduli_divide(tail, m)
        implies all_moduli_divide(List.cons(head, tail), m)
} by {
    if head.first.divides(m) and all_moduli_divide(tail, m) {
        all_moduli_divide(List.cons(head, tail), m)
    }
}

/// The empty system divides anything.
theorem all_moduli_divide_nil(m: Nat) {
    all_moduli_divide(List.nil[Pair[Nat, Nat]], m)
}

/// Congruence descends to a divisor of the modulus.
///
/// The relation is divisibility of the difference by the modulus, so a smaller modulus that
/// divides the larger inherits it.
theorem int_mod_rel_descend(m: Nat, big: Nat, x: Int, y: Int) {
    m.divides(big) and int_mod_rel(big, x, y) implies int_mod_rel(m, x, y)
} by {
    if m.divides(big) and int_mod_rel(big, x, y) {
        int_mod_rel(big, x, y) = Int.from_nat(big).divides(x - y)
        Int.from_nat(big).divides(x - y)
        div_from_nat(m, big)
        Int.from_nat(m).divides(Int.from_nat(big))
        div_trans(Int.from_nat(m), Int.from_nat(big), x - y)
        Int.from_nat(m).divides(x - y)
        int_mod_rel(m, x, y) = Int.from_nat(m).divides(x - y)
        int_mod_rel(m, x, y)
    }
}

/// A congruence class whose modulus divides the period is invariant along it.
theorem class_periodic(cl: Pair[Nat, Nat], m: Nat, x: Int, y: Int) {
    cl.first.divides(m) and int_mod_rel(m, x, y) and congruence_class_contains(cl, y)
        implies congruence_class_contains(cl, x)
} by {
    if cl.first.divides(m) and int_mod_rel(m, x, y) and congruence_class_contains(cl, y) {
        int_mod_rel_descend(cl.first, m, x, y)
        int_mod_rel(cl.first, x, y)
        congruence_class_contains_of_congruent(cl, x, y)
        congruence_class_contains(cl, x)
    }
}

/// A system whose moduli all divide the period is invariant along it.
///
/// This is what makes a covering or disjointness question finite: the answer at an integer is
/// determined by its residue modulo the common period.
theorem system_periodic(
    system: List[Pair[Nat, Nat]], m: Nat, x: Int, y: Int
) {
    all_moduli_divide(system, m) and int_mod_rel(m, x, y) and covers_int(system, y)
        implies covers_int(system, x)
} by {
    define p(s: List[Pair[Nat, Nat]]) -> Bool {
        all_moduli_divide(s, m) and int_mod_rel(m, x, y) and covers_int(s, y)
            implies covers_int(s, x)
    }
    if all_moduli_divide(List.nil[Pair[Nat, Nat]], m) and int_mod_rel(m, x, y)
        and covers_int(List.nil[Pair[Nat, Nat]], y) {
        covers_int_nil_false(y)
        not covers_int(List.nil[Pair[Nat, Nat]], y)
        false
    }
    p(List.nil[Pair[Nat, Nat]])
    forall(head: Pair[Nat, Nat], tail: List[Pair[Nat, Nat]]) {
        if p(tail) {
            if all_moduli_divide(List.cons(head, tail), m) and int_mod_rel(m, x, y)
                and covers_int(List.cons(head, tail), y) {
                all_moduli_divide_head(head, tail, m)
                head.first.divides(m)
                all_moduli_divide_tail(head, tail, m)
                all_moduli_divide(tail, m)
                covers_int_cons_imp(head, tail, y)
                (congruence_class_contains(head, y) or covers_int(tail, y))
                if congruence_class_contains(head, y) {
                    class_periodic(head, m, x, y)
                    congruence_class_contains(head, x)
                    covers_int_cons_left(head, tail, x)
                    covers_int(List.cons(head, tail), x)
                }
                if not congruence_class_contains(head, y) {
                    covers_int(tail, y)
                    covers_int(tail, x)
                    covers_int_cons_right(head, tail, x)
                    covers_int(List.cons(head, tail), x)
                }
                covers_int(List.cons(head, tail), x)
            }
            p(List.cons(head, tail))
        }
        (p(tail) implies p(List.cons(head, tail)))
    }
    p(List.nil[Pair[Nat, Nat]]) and forall(head: Pair[Nat, Nat], tail: List[Pair[Nat, Nat]]) {
        p(tail) implies p(List.cons(head, tail))
    }
    List.induction(p)
    p(system)
}

/// True if every modulus of the system exceeds one.
///
/// The condition that rules out the trivial covering system consisting of the single class
/// of everything modulo one.
define all_moduli_gt_one(system: List[Pair[Nat, Nat]]) -> Bool {
    match system {
        List.nil {
            true
        }
        List.cons(head, tail) {
            Nat.1 < head.first and all_moduli_gt_one(tail)
        }
    }
}

/// The head modulus of such a system exceeds one.
theorem all_moduli_gt_one_head(head: Pair[Nat, Nat], tail: List[Pair[Nat, Nat]]) {
    all_moduli_gt_one(List.cons(head, tail)) implies Nat.1 < head.first
} by {
    if all_moduli_gt_one(List.cons(head, tail)) {
        Nat.1 < head.first
    }
}

/// The tail of such a system satisfies the condition.
theorem all_moduli_gt_one_tail(head: Pair[Nat, Nat], tail: List[Pair[Nat, Nat]]) {
    all_moduli_gt_one(List.cons(head, tail)) implies all_moduli_gt_one(tail)
} by {
    if all_moduli_gt_one(List.cons(head, tail)) {
        all_moduli_gt_one(tail)
    }
}

/// The two conditions give the whole system.
theorem all_moduli_gt_one_cons_intro(head: Pair[Nat, Nat], tail: List[Pair[Nat, Nat]]) {
    Nat.1 < head.first and all_moduli_gt_one(tail)
        implies all_moduli_gt_one(List.cons(head, tail))
} by {
    if Nat.1 < head.first and all_moduli_gt_one(tail) {
        all_moduli_gt_one(List.cons(head, tail))
    }
}

/// True if every modulus of the system is odd.
///
/// One of the standard side conditions on covering systems, alongside the moduli exceeding
/// one and being distinct.
define all_moduli_odd(system: List[Pair[Nat, Nat]]) -> Bool {
    match system {
        List.nil {
            true
        }
        List.cons(head, tail) {
            not Nat.2.divides(head.first) and all_moduli_odd(tail)
        }
    }
}

/// The head modulus of such a system is odd.
theorem all_moduli_odd_head(head: Pair[Nat, Nat], tail: List[Pair[Nat, Nat]]) {
    all_moduli_odd(List.cons(head, tail)) implies not Nat.2.divides(head.first)
} by {
    if all_moduli_odd(List.cons(head, tail)) {
        not Nat.2.divides(head.first)
    }
}

/// The tail of such a system satisfies the condition.
theorem all_moduli_odd_tail(head: Pair[Nat, Nat], tail: List[Pair[Nat, Nat]]) {
    all_moduli_odd(List.cons(head, tail)) implies all_moduli_odd(tail)
} by {
    if all_moduli_odd(List.cons(head, tail)) {
        all_moduli_odd(tail)
    }
}

/// True if some class of the system has the given modulus.
define has_modulus(system: List[Pair[Nat, Nat]], m: Nat) -> Bool {
    match system {
        List.nil {
            false
        }
        List.cons(head, tail) {
            head.first = m or has_modulus(tail, m)
        }
    }
}

/// The empty system has no modulus.
theorem has_modulus_nil(m: Nat) {
    not has_modulus(List.nil[Pair[Nat, Nat]], m)
} by {
    has_modulus(List.nil[Pair[Nat, Nat]], m) = false
}

/// The head supplies its own modulus.
theorem has_modulus_cons_head(head: Pair[Nat, Nat], tail: List[Pair[Nat, Nat]]) {
    has_modulus(List.cons(head, tail), head.first)
}

/// A modulus of the tail is a modulus of the whole.
theorem has_modulus_cons_tail(head: Pair[Nat, Nat], tail: List[Pair[Nat, Nat]], m: Nat) {
    has_modulus(tail, m) implies has_modulus(List.cons(head, tail), m)
} by {
    if has_modulus(tail, m) {
        has_modulus(List.cons(head, tail), m)
    }
}

/// A modulus of a cons system comes from the head or the tail.
theorem has_modulus_cons_imp(head: Pair[Nat, Nat], tail: List[Pair[Nat, Nat]], m: Nat) {
    has_modulus(List.cons(head, tail), m) implies head.first = m or has_modulus(tail, m)
} by {
    if has_modulus(List.cons(head, tail), m) {
        head.first = m or has_modulus(tail, m)
    }
}

/// True if no modulus repeats.
///
/// The distinct-moduli condition, stated recursively against the rest of the list in the same
/// shape as disjointness.
define distinct_moduli(system: List[Pair[Nat, Nat]]) -> Bool {
    match system {
        List.nil {
            true
        }
        List.cons(head, tail) {
            not has_modulus(tail, head.first) and distinct_moduli(tail)
        }
    }
}

/// The head modulus of such a system does not reappear.
theorem distinct_moduli_head(head: Pair[Nat, Nat], tail: List[Pair[Nat, Nat]]) {
    distinct_moduli(List.cons(head, tail)) implies not has_modulus(tail, head.first)
} by {
    if distinct_moduli(List.cons(head, tail)) {
        not has_modulus(tail, head.first)
    }
}

/// The tail of such a system satisfies the condition.
theorem distinct_moduli_tail(head: Pair[Nat, Nat], tail: List[Pair[Nat, Nat]]) {
    distinct_moduli(List.cons(head, tail)) implies distinct_moduli(tail)
} by {
    if distinct_moduli(List.cons(head, tail)) {
        distinct_moduli(tail)
    }
}

/// The two conditions give the whole system.
theorem distinct_moduli_cons_intro(head: Pair[Nat, Nat], tail: List[Pair[Nat, Nat]]) {
    not has_modulus(tail, head.first) and distinct_moduli(tail)
        implies distinct_moduli(List.cons(head, tail))
} by {
    if not has_modulus(tail, head.first) and distinct_moduli(tail) {
        distinct_moduli(List.cons(head, tail))
    }
}
