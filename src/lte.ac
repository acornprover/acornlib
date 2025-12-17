/// The "LTE" typeclass represents anything that has a less-than-or-equal-to relation.
typeclass A: LTE {
    lte: (A, A) -> Bool
}
