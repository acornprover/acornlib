/// Product-vanishing bridges for the Zariski spectrum.
///
/// This focused bridge supplies the product form of the binary Zariski union
/// law using the separately verified ideal-product API.

from comm_ring import CommRing
from algebra.ring.ideal import Ideal, bundled_ideal_subset, bundled_ideal_subset_trans,
    prime_ideal_as_ideal_contains_eq, prime_ideal_contains_mul
from algebra.ring.ideal_product import ideal_product, ideal_product_contains_mul,
    ideal_product_subset_left, ideal_product_subset_right
from algebra.ring.spec import Spec, spec_vanishing_contains_eq, spec_vanishing_union_eq_inter

/// `V(I) ∪ V(J) ⊆ V(IJ)`: if a prime contains either factor, it contains their product.
theorem spec_vanishing_union_subset_product[R: CommRing](i: Ideal[R], j: Ideal[R]) {
    Spec[R].vanishing(i).union(Spec[R].vanishing(j)).subset(
        Spec[R].vanishing(ideal_product(i, j)))
} by {
    ideal_product_subset_left(i, j)
    ideal_product_subset_right(i, j)
    forall(p: Spec[R]) {
        if Spec[R].vanishing(i).union(Spec[R].vanishing(j)).contains(p) {
            if Spec[R].vanishing(i).contains(p) {
                spec_vanishing_contains_eq(i, p)
                bundled_ideal_subset(i, p.point.as_ideal)
                bundled_ideal_subset_trans(ideal_product(i, j), i, p.point.as_ideal)
                bundled_ideal_subset(ideal_product(i, j), p.point.as_ideal)
                spec_vanishing_contains_eq(ideal_product(i, j), p)
                Spec[R].vanishing(ideal_product(i, j)).contains(p)
            }
            if Spec[R].vanishing(j).contains(p) {
                spec_vanishing_contains_eq(j, p)
                bundled_ideal_subset(j, p.point.as_ideal)
                bundled_ideal_subset_trans(ideal_product(i, j), j, p.point.as_ideal)
                bundled_ideal_subset(ideal_product(i, j), p.point.as_ideal)
                spec_vanishing_contains_eq(ideal_product(i, j), p)
                Spec[R].vanishing(ideal_product(i, j)).contains(p)
            }
            Spec[R].vanishing(ideal_product(i, j)).contains(p)
        }
    }
}

/// `V(IJ) ⊆ V(I) ∪ V(J)`: a prime containing all products contains at least one factor.
theorem spec_vanishing_product_subset_union[R: CommRing](i: Ideal[R], j: Ideal[R]) {
    Spec[R].vanishing(ideal_product(i, j)).subset(
        Spec[R].vanishing(i).union(Spec[R].vanishing(j)))
} by {
    forall(p: Spec[R]) {
        if Spec[R].vanishing(ideal_product(i, j)).contains(p) {
            spec_vanishing_contains_eq(ideal_product(i, j), p)
            bundled_ideal_subset(ideal_product(i, j), p.point.as_ideal)
            if not bundled_ideal_subset(i, p.point.as_ideal) {
                if not bundled_ideal_subset(j, p.point.as_ideal) {
                    let a: R satisfy {
                        i.contains(a) and not p.point.as_ideal.contains(a)
                    }
                    let b: R satisfy {
                        j.contains(b) and not p.point.as_ideal.contains(b)
                    }
                    ideal_product_contains_mul(i, j, a, b)
                    ideal_product(i, j).contains(a * b)
                    bundled_ideal_subset(ideal_product(i, j), p.point.as_ideal) = forall(x: R) {
                        ideal_product(i, j).contains(x) implies p.point.as_ideal.contains(x)
                    }
                    p.point.as_ideal.contains(a * b)
                    prime_ideal_as_ideal_contains_eq(p.point, a * b)
                    p.point.contains(a * b)
                    prime_ideal_contains_mul(p.point, a, b)
                    p.point.contains(a) or p.point.contains(b)
                    prime_ideal_as_ideal_contains_eq(p.point, a)
                    prime_ideal_as_ideal_contains_eq(p.point, b)
                    false
                }
                bundled_ideal_subset(j, p.point.as_ideal)
                spec_vanishing_contains_eq(j, p)
                Spec[R].vanishing(j).contains(p)
                Spec[R].vanishing(i).union(Spec[R].vanishing(j)).contains(p)
            }
            if bundled_ideal_subset(i, p.point.as_ideal) {
                spec_vanishing_contains_eq(i, p)
                Spec[R].vanishing(i).contains(p)
                Spec[R].vanishing(i).union(Spec[R].vanishing(j)).contains(p)
            }
            Spec[R].vanishing(i).union(Spec[R].vanishing(j)).contains(p)
        }
    }
}

/// `V(IJ) = V(I) ∪ V(J)`: binary Zariski unions are represented by ideal products.
theorem spec_vanishing_product_eq_union[R: CommRing](i: Ideal[R], j: Ideal[R]) {
    Spec[R].vanishing(ideal_product(i, j)) =
        Spec[R].vanishing(i).union(Spec[R].vanishing(j))
} by {
    spec_vanishing_product_subset_union(i, j)
    spec_vanishing_union_subset_product(i, j)
}

/// `V(IJ) = V(I ∩ J)`: product and intersection define the same vanishing set.
theorem spec_vanishing_product_eq_intersection[R: CommRing](i: Ideal[R], j: Ideal[R]) {
    Spec[R].vanishing(ideal_product(i, j)) = Spec[R].vanishing(i.intersection(j))
} by {
    spec_vanishing_product_eq_union(i, j)
    spec_vanishing_union_eq_inter(i, j)
    Spec[R].vanishing(ideal_product(i, j)) = Spec[R].vanishing(i).union(Spec[R].vanishing(j))
    Spec[R].vanishing(i).union(Spec[R].vanishing(j)) = Spec[R].vanishing(i.intersection(j))
}
