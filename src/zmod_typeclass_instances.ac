from nat import Nat
from int import Int
from zmod import Zmod, zmod_mk, zmod_zero, zmod_one, zmod_add, zmod_neg, zmod_mul,
    zmod_zero_mk, zmod_one_mk, zmod_add_mk, zmod_neg_mk, zmod_mul_mk,
    zmod_add_assoc, zmod_add_comm, zmod_add_zero_left, zmod_add_zero_right,
    zmod_add_neg_right, zmod_mul_assoc, zmod_mul_comm, zmod_mul_one_left,
    zmod_mul_one_right, zmod_mul_zero_left, zmod_mul_zero_right,
    zmod_mul_add_left, zmod_mul_add_right
from algebra.add import Add
from algebra.zero import Zero
from algebra.neg import Neg
from algebra.mul import Mul
from algebra.one import One
from algebra.add_semigroup import AddSemigroup
from algebra.add_comm_semigroup import AddCommSemigroup
from algebra.add_monoid import AddMonoid
from algebra.add_comm_monoid import AddCommMonoid
from algebra.add_group import AddGroup
from algebra.add_comm_group import AddCommGroup
from algebra.semigroup import Semigroup
from algebra.comm_semigroup import CommSemigroup
from algebra.monoid.monoid import Monoid
from algebra.comm_monoid import CommMonoid
from semiring import Semiring
from algebra.ring.ring import Ring
from comm_ring import CommRing

/// The typeclass addition on `Zmod[n]` is the existing residue-class addition.
theorem zmod_typeclass_add_eq(n: Nat, x: Zmod[n], y: Zmod[n]) {
    x + y = zmod_add(n, x, y)
} by {
    x + y = zmod_add(n, x, y)
}

/// The typeclass zero on `Zmod[n]` is the existing zero residue class.
theorem zmod_typeclass_zero_eq(n: Nat) {
    Zero.0[Zmod[n]] = zmod_zero(n)
} by {
    Zero.0[Zmod[n]] = zmod_zero(n)
}

/// The typeclass negation on `Zmod[n]` is the existing residue-class negation.
theorem zmod_typeclass_neg_eq(n: Nat, x: Zmod[n]) {
    -x = zmod_neg(n, x)
} by {
    -x = zmod_neg(n, x)
}

/// The typeclass multiplication on `Zmod[n]` is the existing residue-class multiplication.
theorem zmod_typeclass_mul_eq(n: Nat, x: Zmod[n], y: Zmod[n]) {
    x * y = zmod_mul(n, x, y)
} by {
    x * y = zmod_mul(n, x, y)
}

/// The typeclass one on `Zmod[n]` is the existing one residue class.
theorem zmod_typeclass_one_eq(n: Nat) {
    One.1[Zmod[n]] = zmod_one(n)
} by {
    One.1[Zmod[n]] = zmod_one(n)
}

/// Typeclass addition agrees with addition of integer representatives.
theorem zmod_typeclass_add_mk(n: Nat, a: Int, b: Int) {
    zmod_mk(n, a) + zmod_mk(n, b) = zmod_mk(n, a + b)
} by {
    zmod_typeclass_add_eq(n, zmod_mk(n, a), zmod_mk(n, b))
    zmod_add_mk(n, a, b)
}

/// Typeclass negation agrees with negation of integer representatives.
theorem zmod_typeclass_neg_mk(n: Nat, a: Int) {
    -zmod_mk(n, a) = zmod_mk(n, -a)
} by {
    zmod_typeclass_neg_eq(n, zmod_mk(n, a))
    zmod_neg_mk(n, a)
}

/// Typeclass multiplication agrees with multiplication of integer representatives.
theorem zmod_typeclass_mul_mk(n: Nat, a: Int, b: Int) {
    zmod_mk(n, a) * zmod_mk(n, b) = zmod_mk(n, a * b)
} by {
    zmod_typeclass_mul_eq(n, zmod_mk(n, a), zmod_mk(n, b))
    zmod_mul_mk(n, a, b)
}

/// The typeclass zero is represented by integer zero.
theorem zmod_typeclass_zero_mk(n: Nat) {
    Zero.0[Zmod[n]] = zmod_mk(n, Int.0)
} by {
    zmod_typeclass_zero_eq(n)
    zmod_zero_mk(n)
}

/// The typeclass one is represented by integer one.
theorem zmod_typeclass_one_mk(n: Nat) {
    One.1[Zmod[n]] = zmod_mk(n, Int.1)
} by {
    zmod_typeclass_one_eq(n)
    zmod_one_mk(n)
}

/// Typeclass addition on `Zmod[n]` is associative.
theorem zmod_typeclass_add_associative(n: Nat, x: Zmod[n], y: Zmod[n], z: Zmod[n]) {
    Add.add(x, Add.add(y, z)) = Add.add(Add.add(x, y), z)
} by {
    zmod_typeclass_add_eq(n, y, z)
    zmod_typeclass_add_eq(n, x, zmod_add(n, y, z))
    Add.add(x, Add.add(y, z)) = zmod_add(n, x, zmod_add(n, y, z))
    zmod_typeclass_add_eq(n, x, y)
    zmod_typeclass_add_eq(n, zmod_add(n, x, y), z)
    Add.add(Add.add(x, y), z) = zmod_add(n, zmod_add(n, x, y), z)
    zmod_add_assoc(n, x, y, z)
    zmod_add(n, x, zmod_add(n, y, z)) = zmod_add(n, zmod_add(n, x, y), z)
}

/// Typeclass addition on `Zmod[n]` is commutative.
theorem zmod_typeclass_add_commutative(n: Nat, x: Zmod[n], y: Zmod[n]) {
    Add.add(x, y) = Add.add(y, x)
} by {
    zmod_typeclass_add_eq(n, x, y)
    zmod_typeclass_add_eq(n, y, x)
    zmod_add_comm(n, x, y)
}

/// The typeclass zero is a left identity for addition on `Zmod[n]`.
theorem zmod_typeclass_add_zero_left(n: Nat, x: Zmod[n]) {
    Add.add(Zero.0[Zmod[n]], x) = x
} by {
    zmod_typeclass_zero_eq(n)
    zmod_typeclass_add_eq(n, zmod_zero(n), x)
    zmod_add_zero_left(n, x)
}

/// The typeclass zero is a right identity for addition on `Zmod[n]`.
theorem zmod_typeclass_add_zero_right(n: Nat, x: Zmod[n]) {
    Add.add(x, Zero.0[Zmod[n]]) = x
} by {
    zmod_typeclass_zero_eq(n)
    zmod_typeclass_add_eq(n, x, zmod_zero(n))
    zmod_add_zero_right(n, x)
}

/// The typeclass negation is a right additive inverse on `Zmod[n]`.
theorem zmod_typeclass_add_neg_right(n: Nat, x: Zmod[n]) {
    Add.add(x, Neg.neg(x)) = Zero.0[Zmod[n]]
} by {
    zmod_typeclass_neg_eq(n, x)
    zmod_typeclass_zero_eq(n)
    zmod_typeclass_add_eq(n, x, zmod_neg(n, x))
    zmod_add_neg_right(n, x)
}

/// Typeclass multiplication on `Zmod[n]` is associative.
theorem zmod_typeclass_mul_associative(n: Nat, x: Zmod[n], y: Zmod[n], z: Zmod[n]) {
    Mul.mul(x, Mul.mul(y, z)) = Mul.mul(Mul.mul(x, y), z)
} by {
    zmod_typeclass_mul_eq(n, y, z)
    zmod_typeclass_mul_eq(n, x, zmod_mul(n, y, z))
    Mul.mul(x, Mul.mul(y, z)) = zmod_mul(n, x, zmod_mul(n, y, z))
    zmod_typeclass_mul_eq(n, x, y)
    zmod_typeclass_mul_eq(n, zmod_mul(n, x, y), z)
    Mul.mul(Mul.mul(x, y), z) = zmod_mul(n, zmod_mul(n, x, y), z)
    zmod_mul_assoc(n, x, y, z)
    zmod_mul(n, x, zmod_mul(n, y, z)) = zmod_mul(n, zmod_mul(n, x, y), z)
}

/// Typeclass multiplication on `Zmod[n]` is commutative.
theorem zmod_typeclass_mul_commutative(n: Nat, x: Zmod[n], y: Zmod[n]) {
    Mul.mul(x, y) = Mul.mul(y, x)
} by {
    zmod_typeclass_mul_eq(n, x, y)
    zmod_typeclass_mul_eq(n, y, x)
    zmod_mul_comm(n, x, y)
}

/// The typeclass one is a left identity for multiplication on `Zmod[n]`.
theorem zmod_typeclass_mul_one_left(n: Nat, x: Zmod[n]) {
    Mul.mul(One.1[Zmod[n]], x) = x
} by {
    zmod_typeclass_one_eq(n)
    zmod_typeclass_mul_eq(n, zmod_one(n), x)
    zmod_mul_one_left(n, x)
}

/// The typeclass one is a right identity for multiplication on `Zmod[n]`.
theorem zmod_typeclass_mul_one_right(n: Nat, x: Zmod[n]) {
    Mul.mul(x, One.1[Zmod[n]]) = x
} by {
    zmod_typeclass_one_eq(n)
    zmod_typeclass_mul_eq(n, x, zmod_one(n))
    zmod_mul_one_right(n, x)
}

/// Typeclass multiplication distributes on the left over typeclass addition on `Zmod[n]`.
theorem zmod_typeclass_mul_add_left(n: Nat, x: Zmod[n], y: Zmod[n], z: Zmod[n]) {
    Mul.mul(x, Add.add(y, z)) = Add.add(Mul.mul(x, y), Mul.mul(x, z))
} by {
    zmod_typeclass_add_eq(n, y, z)
    zmod_typeclass_mul_eq(n, x, zmod_add(n, y, z))
    Mul.mul(x, Add.add(y, z)) = zmod_mul(n, x, zmod_add(n, y, z))
    zmod_typeclass_mul_eq(n, x, y)
    zmod_typeclass_mul_eq(n, x, z)
    zmod_typeclass_add_eq(n, zmod_mul(n, x, y), zmod_mul(n, x, z))
    Add.add(Mul.mul(x, y), Mul.mul(x, z)) =
        zmod_add(n, zmod_mul(n, x, y), zmod_mul(n, x, z))
    zmod_mul_add_left(n, x, y, z)
}

/// Typeclass multiplication distributes on the right over typeclass addition on `Zmod[n]`.
theorem zmod_typeclass_mul_add_right(n: Nat, x: Zmod[n], y: Zmod[n], z: Zmod[n]) {
    Mul.mul(Add.add(x, y), z) = Add.add(Mul.mul(x, z), Mul.mul(y, z))
} by {
    zmod_typeclass_add_eq(n, x, y)
    zmod_typeclass_mul_eq(n, zmod_add(n, x, y), z)
    Mul.mul(Add.add(x, y), z) = zmod_mul(n, zmod_add(n, x, y), z)
    zmod_typeclass_mul_eq(n, x, z)
    zmod_typeclass_mul_eq(n, y, z)
    zmod_typeclass_add_eq(n, zmod_mul(n, x, z), zmod_mul(n, y, z))
    Add.add(Mul.mul(x, z), Mul.mul(y, z)) =
        zmod_add(n, zmod_mul(n, x, z), zmod_mul(n, y, z))
    zmod_mul_add_right(n, x, y, z)
}

/// Multiplying on the right by the typeclass zero gives the typeclass zero on `Zmod[n]`.
theorem zmod_typeclass_mul_zero_right(n: Nat, x: Zmod[n]) {
    Mul.mul(x, Zero.0[Zmod[n]]) = Zero.0[Zmod[n]]
} by {
    zmod_typeclass_zero_eq(n)
    zmod_typeclass_mul_eq(n, x, zmod_zero(n))
    zmod_mul_zero_right(n, x)
}

/// Multiplying on the left by the typeclass zero gives the typeclass zero on `Zmod[n]`.
theorem zmod_typeclass_mul_zero_left(n: Nat, x: Zmod[n]) {
    Mul.mul(Zero.0[Zmod[n]], x) = Zero.0[Zmod[n]]
} by {
    zmod_typeclass_zero_eq(n)
    zmod_typeclass_mul_eq(n, zmod_zero(n), x)
    zmod_mul_zero_left(n, x)
}

/// Residue classes modulo `n` form an additive semigroup.
instance Zmod[n: Nat]: AddSemigroup

/// Residue classes modulo `n` form an additive commutative semigroup.
instance Zmod[n: Nat]: AddCommSemigroup

/// Residue classes modulo `n` form an additive monoid.
instance Zmod[n: Nat]: AddMonoid

/// Residue classes modulo `n` form an additive commutative monoid.
instance Zmod[n: Nat]: AddCommMonoid

/// Residue classes modulo `n` form an additive group.
instance Zmod[n: Nat]: AddGroup

/// Residue classes modulo `n` form an additive commutative group.
instance Zmod[n: Nat]: AddCommGroup

/// Residue classes modulo `n` form a multiplicative semigroup.
instance Zmod[n: Nat]: Semigroup

/// Residue classes modulo `n` form a multiplicative commutative semigroup.
instance Zmod[n: Nat]: CommSemigroup

/// Residue classes modulo `n` form a multiplicative monoid.
instance Zmod[n: Nat]: Monoid

/// Residue classes modulo `n` form a multiplicative commutative monoid.
instance Zmod[n: Nat]: CommMonoid

/// Residue classes modulo `n` form a semiring.
instance Zmod[n: Nat]: Semiring

/// Residue classes modulo `n` form a ring.
instance Zmod[n: Nat]: Ring

/// Residue classes modulo `n` form a commutative ring.
instance Zmod[n: Nat]: CommRing
