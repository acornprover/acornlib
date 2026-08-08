from algebra.add_comm_group import AddCommGroup
from pair import Pair
from affine_space import AffineSpace, affine_space_product
from affine_subspace import affine_subspace_subset, affine_subspace_product,
    affine_subspace_product_space, affine_subspace_product_contains_iff,
    affine_span, affine_span_space, affine_span_contains_src,
    affine_span_subset

/// True when each coordinate of a point belongs to its corresponding source
/// predicate.
define affine_span_product_src[P, Q](
    src1: P -> Bool, src2: Q -> Bool, x: Pair[P, Q]) -> Bool {
    src1(x.first) and src2(x.second)
}

/// Membership in a product source predicate is coordinatewise membership.
theorem affine_span_product_src_iff[P, Q](
    src1: P -> Bool, src2: Q -> Bool, x: Pair[P, Q]
) {
    affine_span_product_src(src1, src2, x) =
        (src1(x.first) and src2(x.second))
}

/// Every point of a product source predicate belongs to the product of the
/// corresponding affine spans.
theorem affine_span_product_src_mem_product_spans[V: AddCommGroup, W: AddCommGroup, P, Q](
    a: AffineSpace[V, P],
    b: AffineSpace[W, Q],
    src1: P -> Bool,
    src2: Q -> Bool,
    x: Pair[P, Q]
) {
    affine_span_product_src(src1, src2, x) implies
        affine_subspace_product(
            affine_span(a, src1), affine_span(b, src2)).contains(x)
} by {
    if affine_span_product_src(src1, src2, x) {
        let span1 = affine_span(a, src1)
        let span2 = affine_span(b, src2)
        let product = affine_subspace_product(span1, span2)
        affine_span_product_src_iff(src1, src2, x)
        src1(x.first)
        src2(x.second)
        affine_span_contains_src(a, src1, x.first)
        span1.contains(x.first)
        affine_span_contains_src(b, src2, x.second)
        span2.contains(x.second)
        affine_subspace_product_contains_iff(
            span1, span2, x)
        product.contains(x) =
            (span1.contains(x.first) and span2.contains(x.second))
        product.contains(x)
        affine_subspace_product(
            affine_span(a, src1), affine_span(b, src2)).contains(x)
    }
}

/// The affine span of a product source predicate is contained in the product
/// of the corresponding affine spans.
theorem affine_span_product_subset_product_spans[V: AddCommGroup, W: AddCommGroup, P, Q](
    a: AffineSpace[V, P],
    b: AffineSpace[W, Q],
    src1: P -> Bool,
    src2: Q -> Bool
) {
    affine_subspace_subset(
        affine_span(
            affine_space_product(a, b),
            affine_span_product_src(src1, src2)),
        affine_subspace_product(
            affine_span(a, src1), affine_span(b, src2)))
} by {
    let span1 = affine_span(a, src1)
    let span2 = affine_span(b, src2)
    let product = affine_subspace_product(span1, span2)
    affine_span_space(a, src1)
    affine_span_space(b, src2)
    affine_subspace_product_space(span1, span2)
    product.space = affine_space_product(a, b)
    forall(x: Pair[P, Q]) {
        if affine_span_product_src(src1, src2, x) {
            affine_span_product_src_mem_product_spans(
                a, b, src1, src2, x)
            product.contains(x)
        }
    }
    affine_span_subset(
        affine_space_product(a, b),
        affine_span_product_src(src1, src2),
        product)
    affine_subspace_subset(
        affine_span(
            affine_space_product(a, b),
            affine_span_product_src(src1, src2)),
        product)
    affine_subspace_subset(
        affine_span(
            affine_space_product(a, b),
            affine_span_product_src(src1, src2)),
        affine_subspace_product(
            affine_span(a, src1), affine_span(b, src2)))
}
