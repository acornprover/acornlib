/// Transport of order bounds and extrema through monotone and antitone maps.
///
/// This module bridges the predicate-level bound API in `order_bounds` and the
/// function-on-domain bound API in `order_function_bounds` with the reusable
/// `is_monotone`, `is_antitone`, and `is_order_embedding` map interface.

from data.basic.functions import compose, is_surjective_fn, surjective_fn_has_preimage
from order import PartialOrder, is_monotone, is_antitone, is_order_embedding,
    monotone_apply, antitone_apply, order_embedding_is_monotone,
    order_embedding_apply_le, order_embedding_reflects_lte
from order_bounds import is_lower_bound, is_upper_bound, is_least, is_greatest,
    is_glb, is_lub, is_bounded_below, is_bounded_above,
    lower_bound_le, upper_bound_ge, is_least_mem, is_greatest_mem,
    is_glb_ge_lower_bound, is_lub_le_upper_bound
from order_function_bounds import is_upper_bound_on, is_lower_bound_on,
    is_bounded_above_on, is_bounded_below_on, attains_maximum_at,
    attains_minimum_at, attains_maximum_on, attains_minimum_on,
    upper_bound_on_apply, lower_bound_on_apply, maximum_point_in_domain,
    minimum_point_in_domain
from data.basic.set import Set

/// The direct image of a predicate under a map.
define map_image_predicate[A, B](f: A -> B, p: A -> Bool, y: B) -> Bool {
    exists(x: A) {
        p(x) and y = f(x)
    }
}

/// The preimage of a predicate under a map.
define map_preimage_predicate[A, B](f: A -> B, q: B -> Bool, x: A) -> Bool {
    q(f(x))
}

/// Every member of a predicate maps into its direct image predicate.
theorem map_image_predicate_of[A, B](f: A -> B, p: A -> Bool, x: A) {
    p(x) implies map_image_predicate(f, p, f(x))
} by {
    if p(x) {
        exists(w: A) {
            w = x and p(w) and f(x) = f(w)
        }
        map_image_predicate(f, p, f(x))
    }
}

/// A member of a direct image predicate has a preimage witness.
theorem map_image_predicate_witness[A, B](f: A -> B, p: A -> Bool, y: B) {
    map_image_predicate(f, p, y) implies exists(x: A) {
        p(x) and y = f(x)
    }
} by {
    if map_image_predicate(f, p, y) {
        map_image_predicate(f, p, y) = exists(x: A) {
            p(x) and y = f(x)
        }
    }
}

/// A monotone map carries a lower bound to a lower bound of the direct image.
theorem monotone_image_lower_bound[A: PartialOrder, B: PartialOrder](
    f: A -> B,
    p: A -> Bool,
    a: A
) {
    is_monotone(f) and is_lower_bound(p, a)
    implies is_lower_bound(map_image_predicate(f, p), f(a))
} by {
    if is_monotone(f) and is_lower_bound(p, a) {
        forall(y: B) {
            if map_image_predicate(f, p, y) {
                map_image_predicate_witness(f, p, y)
                let x: A satisfy {
                    p(x) and y = f(x)
                }
                lower_bound_le(p, a, x)
                a <= x
                monotone_apply(f, a, x)
                f(a) <= f(x)
                y = f(x)
                f(a) <= y
            }
        }
    }
}

/// A monotone map carries an upper bound to an upper bound of the direct image.
theorem monotone_image_upper_bound[A: PartialOrder, B: PartialOrder](
    f: A -> B,
    p: A -> Bool,
    b: A
) {
    is_monotone(f) and is_upper_bound(p, b)
    implies is_upper_bound(map_image_predicate(f, p), f(b))
} by {
    if is_monotone(f) and is_upper_bound(p, b) {
        forall(y: B) {
            if map_image_predicate(f, p, y) {
                map_image_predicate_witness(f, p, y)
                let x: A satisfy {
                    p(x) and y = f(x)
                }
                upper_bound_ge(p, b, x)
                x <= b
                monotone_apply(f, x, b)
                f(x) <= f(b)
                y = f(x)
                y <= f(b)
            }
        }
    }
}

/// An antitone map carries a lower bound to an upper bound of the direct image.
theorem antitone_image_lower_bound_to_upper_bound[A: PartialOrder, B: PartialOrder](
    f: A -> B,
    p: A -> Bool,
    a: A
) {
    is_antitone(f) and is_lower_bound(p, a)
    implies is_upper_bound(map_image_predicate(f, p), f(a))
} by {
    if is_antitone(f) and is_lower_bound(p, a) {
        forall(y: B) {
            if map_image_predicate(f, p, y) {
                map_image_predicate_witness(f, p, y)
                let x: A satisfy {
                    p(x) and y = f(x)
                }
                lower_bound_le(p, a, x)
                a <= x
                antitone_apply(f, a, x)
                f(x) <= f(a)
                y = f(x)
                y <= f(a)
            }
        }
    }
}

/// An antitone map carries an upper bound to a lower bound of the direct image.
theorem antitone_image_upper_bound_to_lower_bound[A: PartialOrder, B: PartialOrder](
    f: A -> B,
    p: A -> Bool,
    b: A
) {
    is_antitone(f) and is_upper_bound(p, b)
    implies is_lower_bound(map_image_predicate(f, p), f(b))
} by {
    if is_antitone(f) and is_upper_bound(p, b) {
        forall(y: B) {
            if map_image_predicate(f, p, y) {
                map_image_predicate_witness(f, p, y)
                let x: A satisfy {
                    p(x) and y = f(x)
                }
                upper_bound_ge(p, b, x)
                x <= b
                antitone_apply(f, x, b)
                f(b) <= f(x)
                y = f(x)
                f(b) <= y
            }
        }
    }
}

/// A monotone map carries least elements to least elements of direct images.
theorem monotone_image_least[A: PartialOrder, B: PartialOrder](
    f: A -> B,
    p: A -> Bool,
    a: A
) {
    is_monotone(f) and is_least(p, a)
    implies is_least(map_image_predicate(f, p), f(a))
} by {
    if is_monotone(f) and is_least(p, a) {
        is_least_mem(p, a)
        p(a)
        map_image_predicate_of(f, p, a)
        map_image_predicate(f, p, f(a))
        is_lower_bound(p, a)
        monotone_image_lower_bound(f, p, a)
        is_lower_bound(map_image_predicate(f, p), f(a))
        is_least(map_image_predicate(f, p), f(a))
    }
}

/// A monotone map carries greatest elements to greatest elements of direct images.
theorem monotone_image_greatest[A: PartialOrder, B: PartialOrder](
    f: A -> B,
    p: A -> Bool,
    b: A
) {
    is_monotone(f) and is_greatest(p, b)
    implies is_greatest(map_image_predicate(f, p), f(b))
} by {
    if is_monotone(f) and is_greatest(p, b) {
        is_greatest_mem(p, b)
        p(b)
        map_image_predicate_of(f, p, b)
        map_image_predicate(f, p, f(b))
        is_upper_bound(p, b)
        monotone_image_upper_bound(f, p, b)
        is_upper_bound(map_image_predicate(f, p), f(b))
        is_greatest(map_image_predicate(f, p), f(b))
    }
}

/// An antitone map carries least elements to greatest elements of direct images.
theorem antitone_image_least_to_greatest[A: PartialOrder, B: PartialOrder](
    f: A -> B,
    p: A -> Bool,
    a: A
) {
    is_antitone(f) and is_least(p, a)
    implies is_greatest(map_image_predicate(f, p), f(a))
} by {
    if is_antitone(f) and is_least(p, a) {
        is_least_mem(p, a)
        p(a)
        map_image_predicate_of(f, p, a)
        map_image_predicate(f, p, f(a))
        is_lower_bound(p, a)
        antitone_image_lower_bound_to_upper_bound(f, p, a)
        is_upper_bound(map_image_predicate(f, p), f(a))
        is_greatest(map_image_predicate(f, p), f(a))
    }
}

/// An antitone map carries greatest elements to least elements of direct images.
theorem antitone_image_greatest_to_least[A: PartialOrder, B: PartialOrder](
    f: A -> B,
    p: A -> Bool,
    b: A
) {
    is_antitone(f) and is_greatest(p, b)
    implies is_least(map_image_predicate(f, p), f(b))
} by {
    if is_antitone(f) and is_greatest(p, b) {
        is_greatest_mem(p, b)
        p(b)
        map_image_predicate_of(f, p, b)
        map_image_predicate(f, p, f(b))
        is_upper_bound(p, b)
        antitone_image_upper_bound_to_lower_bound(f, p, b)
        is_lower_bound(map_image_predicate(f, p), f(b))
        is_least(map_image_predicate(f, p), f(b))
    }
}

/// A monotone map carries bounded-below predicates to bounded-below direct images.
theorem monotone_image_bounded_below[A: PartialOrder, B: PartialOrder](
    f: A -> B,
    p: A -> Bool
) {
    is_monotone(f) and is_bounded_below(p)
    implies is_bounded_below(map_image_predicate(f, p))
} by {
    if is_monotone(f) and is_bounded_below(p) {
        let a: A satisfy {
            is_lower_bound(p, a)
        }
        monotone_image_lower_bound(f, p, a)
        is_lower_bound(map_image_predicate(f, p), f(a))
        exists(witness: B) {
            is_lower_bound(map_image_predicate(f, p), witness)
        }
        is_bounded_below(map_image_predicate(f, p))
    }
}

/// A monotone map carries bounded-above predicates to bounded-above direct images.
theorem monotone_image_bounded_above[A: PartialOrder, B: PartialOrder](
    f: A -> B,
    p: A -> Bool
) {
    is_monotone(f) and is_bounded_above(p)
    implies is_bounded_above(map_image_predicate(f, p))
} by {
    if is_monotone(f) and is_bounded_above(p) {
        let b: A satisfy {
            is_upper_bound(p, b)
        }
        monotone_image_upper_bound(f, p, b)
        is_upper_bound(map_image_predicate(f, p), f(b))
        exists(witness: B) {
            is_upper_bound(map_image_predicate(f, p), witness)
        }
        is_bounded_above(map_image_predicate(f, p))
    }
}

/// An antitone map carries bounded-below predicates to bounded-above direct images.
theorem antitone_image_bounded_below_to_bounded_above[A: PartialOrder, B: PartialOrder](
    f: A -> B,
    p: A -> Bool
) {
    is_antitone(f) and is_bounded_below(p)
    implies is_bounded_above(map_image_predicate(f, p))
} by {
    if is_antitone(f) and is_bounded_below(p) {
        let a: A satisfy {
            is_lower_bound(p, a)
        }
        antitone_image_lower_bound_to_upper_bound(f, p, a)
        is_upper_bound(map_image_predicate(f, p), f(a))
        exists(witness: B) {
            is_upper_bound(map_image_predicate(f, p), witness)
        }
        is_bounded_above(map_image_predicate(f, p))
    }
}

/// An antitone map carries bounded-above predicates to bounded-below direct images.
theorem antitone_image_bounded_above_to_bounded_below[A: PartialOrder, B: PartialOrder](
    f: A -> B,
    p: A -> Bool
) {
    is_antitone(f) and is_bounded_above(p)
    implies is_bounded_below(map_image_predicate(f, p))
} by {
    if is_antitone(f) and is_bounded_above(p) {
        let b: A satisfy {
            is_upper_bound(p, b)
        }
        antitone_image_upper_bound_to_lower_bound(f, p, b)
        is_lower_bound(map_image_predicate(f, p), f(b))
        exists(witness: B) {
            is_lower_bound(map_image_predicate(f, p), witness)
        }
        is_bounded_below(map_image_predicate(f, p))
    }
}

/// A surjective order embedding carries an infimum to the infimum of the direct image.
theorem surjective_order_embedding_image_glb[A: PartialOrder, B: PartialOrder](
    f: A -> B,
    p: A -> Bool,
    a: A
) {
    is_order_embedding(f) and is_surjective_fn(f) and is_glb(p, a)
    implies is_glb(map_image_predicate(f, p), f(a))
} by {
    if is_order_embedding(f) and is_surjective_fn(f) and is_glb(p, a) {
        order_embedding_is_monotone(f)
        is_monotone(f)
        is_lower_bound(p, a)
        monotone_image_lower_bound(f, p, a)
        is_lower_bound(map_image_predicate(f, p), f(a))
        forall(c: B) {
            if is_lower_bound(map_image_predicate(f, p), c) {
                surjective_fn_has_preimage(f, c)
                let x: A satisfy {
                    f(x) = c
                }
                forall(y: A) {
                    if p(y) {
                        map_image_predicate_of(f, p, y)
                        map_image_predicate(f, p, f(y))
                        lower_bound_le(map_image_predicate(f, p), c, f(y))
                        c <= f(y)
                        f(x) = c
                        f(x) <= f(y)
                        order_embedding_reflects_lte(f, x, y)
                        x <= y
                    }
                }
                is_lower_bound(p, x)
                is_glb_ge_lower_bound(p, a, x)
                x <= a
                order_embedding_apply_le(f, x, a)
                f(x) <= f(a)
                f(x) = c
                c <= f(a)
            }
        }
        is_glb(map_image_predicate(f, p), f(a))
    }
}

/// A surjective order embedding carries a supremum to the supremum of the direct image.
theorem surjective_order_embedding_image_lub[A: PartialOrder, B: PartialOrder](
    f: A -> B,
    p: A -> Bool,
    b: A
) {
    is_order_embedding(f) and is_surjective_fn(f) and is_lub(p, b)
    implies is_lub(map_image_predicate(f, p), f(b))
} by {
    if is_order_embedding(f) and is_surjective_fn(f) and is_lub(p, b) {
        order_embedding_is_monotone(f)
        is_monotone(f)
        is_upper_bound(p, b)
        monotone_image_upper_bound(f, p, b)
        is_upper_bound(map_image_predicate(f, p), f(b))
        forall(c: B) {
            if is_upper_bound(map_image_predicate(f, p), c) {
                surjective_fn_has_preimage(f, c)
                let x: A satisfy {
                    f(x) = c
                }
                forall(y: A) {
                    if p(y) {
                        map_image_predicate_of(f, p, y)
                        map_image_predicate(f, p, f(y))
                        upper_bound_ge(map_image_predicate(f, p), c, f(y))
                        f(y) <= c
                        f(x) = c
                        f(y) <= f(x)
                        order_embedding_reflects_lte(f, y, x)
                        y <= x
                    }
                }
                is_upper_bound(p, x)
                is_lub_le_upper_bound(p, b, x)
                b <= x
                order_embedding_apply_le(f, b, x)
                f(b) <= f(x)
                f(x) = c
                f(b) <= c
            }
        }
        is_lub(map_image_predicate(f, p), f(b))
    }
}

/// Postcomposition by a monotone map sends an upper bound to an upper bound.
theorem monotone_compose_upper_bound_on[Domain, Value: PartialOrder, Target: PartialOrder](
    domain: Set[Domain],
    f: Domain -> Value,
    g: Value -> Target,
    bound: Value
) {
    is_monotone(g) and is_upper_bound_on(domain, f, bound)
    implies is_upper_bound_on(domain, compose(g, f), g(bound))
} by {
    if is_monotone(g) and is_upper_bound_on(domain, f, bound) {
        forall(point: Domain) {
            if domain.contains(point) {
                upper_bound_on_apply(domain, f, bound, point)
                f(point) <= bound
                monotone_apply(g, f(point), bound)
                g(f(point)) <= g(bound)
                compose(g, f, point) = g(f(point))
                compose(g, f)(point) <= g(bound)
            }
        }
    }
}

/// Postcomposition by a monotone map sends a lower bound to a lower bound.
theorem monotone_compose_lower_bound_on[Domain, Value: PartialOrder, Target: PartialOrder](
    domain: Set[Domain],
    f: Domain -> Value,
    g: Value -> Target,
    bound: Value
) {
    is_monotone(g) and is_lower_bound_on(domain, f, bound)
    implies is_lower_bound_on(domain, compose(g, f), g(bound))
} by {
    if is_monotone(g) and is_lower_bound_on(domain, f, bound) {
        forall(point: Domain) {
            if domain.contains(point) {
                lower_bound_on_apply(domain, f, bound, point)
                bound <= f(point)
                monotone_apply(g, bound, f(point))
                g(bound) <= g(f(point))
                compose(g, f, point) = g(f(point))
                g(bound) <= compose(g, f)(point)
            }
        }
    }
}

/// Postcomposition by an antitone map sends an upper bound to a lower bound.
theorem antitone_compose_upper_bound_to_lower_bound_on[Domain, Value: PartialOrder, Target: PartialOrder](
    domain: Set[Domain],
    f: Domain -> Value,
    g: Value -> Target,
    bound: Value
) {
    is_antitone(g) and is_upper_bound_on(domain, f, bound)
    implies is_lower_bound_on(domain, compose(g, f), g(bound))
} by {
    if is_antitone(g) and is_upper_bound_on(domain, f, bound) {
        forall(point: Domain) {
            if domain.contains(point) {
                upper_bound_on_apply(domain, f, bound, point)
                f(point) <= bound
                antitone_apply(g, f(point), bound)
                g(bound) <= g(f(point))
                compose(g, f, point) = g(f(point))
                g(bound) <= compose(g, f)(point)
            }
        }
    }
}

/// Postcomposition by an antitone map sends a lower bound to an upper bound.
theorem antitone_compose_lower_bound_to_upper_bound_on[Domain, Value: PartialOrder, Target: PartialOrder](
    domain: Set[Domain],
    f: Domain -> Value,
    g: Value -> Target,
    bound: Value
) {
    is_antitone(g) and is_lower_bound_on(domain, f, bound)
    implies is_upper_bound_on(domain, compose(g, f), g(bound))
} by {
    if is_antitone(g) and is_lower_bound_on(domain, f, bound) {
        forall(point: Domain) {
            if domain.contains(point) {
                lower_bound_on_apply(domain, f, bound, point)
                bound <= f(point)
                antitone_apply(g, bound, f(point))
                g(f(point)) <= g(bound)
                compose(g, f, point) = g(f(point))
                compose(g, f)(point) <= g(bound)
            }
        }
    }
}

/// Postcomposition by a monotone map preserves boundedness above.
theorem monotone_compose_bounded_above_on[Domain, Value: PartialOrder, Target: PartialOrder](
    domain: Set[Domain],
    f: Domain -> Value,
    g: Value -> Target
) {
    is_monotone(g) and is_bounded_above_on(domain, f)
    implies is_bounded_above_on(domain, compose(g, f))
} by {
    if is_monotone(g) and is_bounded_above_on(domain, f) {
        let bound: Value satisfy {
            is_upper_bound_on(domain, f, bound)
        }
        monotone_compose_upper_bound_on(domain, f, g, bound)
        is_upper_bound_on(domain, compose(g, f), g(bound))
        exists(witness: Target) {
            is_upper_bound_on(domain, compose(g, f), witness)
        }
        is_bounded_above_on(domain, compose(g, f))
    }
}

/// Postcomposition by a monotone map preserves boundedness below.
theorem monotone_compose_bounded_below_on[Domain, Value: PartialOrder, Target: PartialOrder](
    domain: Set[Domain],
    f: Domain -> Value,
    g: Value -> Target
) {
    is_monotone(g) and is_bounded_below_on(domain, f)
    implies is_bounded_below_on(domain, compose(g, f))
} by {
    if is_monotone(g) and is_bounded_below_on(domain, f) {
        let bound: Value satisfy {
            is_lower_bound_on(domain, f, bound)
        }
        monotone_compose_lower_bound_on(domain, f, g, bound)
        is_lower_bound_on(domain, compose(g, f), g(bound))
        exists(witness: Target) {
            is_lower_bound_on(domain, compose(g, f), witness)
        }
        is_bounded_below_on(domain, compose(g, f))
    }
}

/// Postcomposition by an antitone map sends boundedness above to boundedness below.
theorem antitone_compose_bounded_above_to_bounded_below_on[Domain, Value: PartialOrder, Target: PartialOrder](
    domain: Set[Domain],
    f: Domain -> Value,
    g: Value -> Target
) {
    is_antitone(g) and is_bounded_above_on(domain, f)
    implies is_bounded_below_on(domain, compose(g, f))
} by {
    if is_antitone(g) and is_bounded_above_on(domain, f) {
        let bound: Value satisfy {
            is_upper_bound_on(domain, f, bound)
        }
        antitone_compose_upper_bound_to_lower_bound_on(domain, f, g, bound)
        is_lower_bound_on(domain, compose(g, f), g(bound))
        exists(witness: Target) {
            is_lower_bound_on(domain, compose(g, f), witness)
        }
        is_bounded_below_on(domain, compose(g, f))
    }
}

/// Postcomposition by an antitone map sends boundedness below to boundedness above.
theorem antitone_compose_bounded_below_to_bounded_above_on[Domain, Value: PartialOrder, Target: PartialOrder](
    domain: Set[Domain],
    f: Domain -> Value,
    g: Value -> Target
) {
    is_antitone(g) and is_bounded_below_on(domain, f)
    implies is_bounded_above_on(domain, compose(g, f))
} by {
    if is_antitone(g) and is_bounded_below_on(domain, f) {
        let bound: Value satisfy {
            is_lower_bound_on(domain, f, bound)
        }
        antitone_compose_lower_bound_to_upper_bound_on(domain, f, g, bound)
        is_upper_bound_on(domain, compose(g, f), g(bound))
        exists(witness: Target) {
            is_upper_bound_on(domain, compose(g, f), witness)
        }
        is_bounded_above_on(domain, compose(g, f))
    }
}

/// Postcomposition by a monotone map preserves an attained maximum point.
theorem monotone_compose_attains_maximum_at[Domain, Value: PartialOrder, Target: PartialOrder](
    domain: Set[Domain],
    f: Domain -> Value,
    g: Value -> Target,
    point: Domain
) {
    is_monotone(g) and attains_maximum_at(domain, f, point)
    implies attains_maximum_at(domain, compose(g, f), point)
} by {
    if is_monotone(g) and attains_maximum_at(domain, f, point) {
        maximum_point_in_domain(domain, f, point)
        domain.contains(point)
        is_upper_bound_on(domain, f, f(point))
        monotone_compose_upper_bound_on(domain, f, g, f(point))
        is_upper_bound_on(domain, compose(g, f), g(f(point)))
        compose(g, f, point) = g(f(point))
        is_upper_bound_on(domain, compose(g, f), compose(g, f)(point))
        attains_maximum_at(domain, compose(g, f), point)
    }
}

/// Postcomposition by a monotone map preserves an attained minimum point.
theorem monotone_compose_attains_minimum_at[Domain, Value: PartialOrder, Target: PartialOrder](
    domain: Set[Domain],
    f: Domain -> Value,
    g: Value -> Target,
    point: Domain
) {
    is_monotone(g) and attains_minimum_at(domain, f, point)
    implies attains_minimum_at(domain, compose(g, f), point)
} by {
    if is_monotone(g) and attains_minimum_at(domain, f, point) {
        minimum_point_in_domain(domain, f, point)
        domain.contains(point)
        is_lower_bound_on(domain, f, f(point))
        monotone_compose_lower_bound_on(domain, f, g, f(point))
        is_lower_bound_on(domain, compose(g, f), g(f(point)))
        compose(g, f, point) = g(f(point))
        is_lower_bound_on(domain, compose(g, f), compose(g, f)(point))
        attains_minimum_at(domain, compose(g, f), point)
    }
}

/// Postcomposition by an antitone map turns an attained maximum point into an attained minimum point.
theorem antitone_compose_attains_maximum_to_minimum_at[Domain, Value: PartialOrder, Target: PartialOrder](
    domain: Set[Domain],
    f: Domain -> Value,
    g: Value -> Target,
    point: Domain
) {
    is_antitone(g) and attains_maximum_at(domain, f, point)
    implies attains_minimum_at(domain, compose(g, f), point)
} by {
    if is_antitone(g) and attains_maximum_at(domain, f, point) {
        maximum_point_in_domain(domain, f, point)
        domain.contains(point)
        is_upper_bound_on(domain, f, f(point))
        antitone_compose_upper_bound_to_lower_bound_on(domain, f, g, f(point))
        is_lower_bound_on(domain, compose(g, f), g(f(point)))
        compose(g, f, point) = g(f(point))
        is_lower_bound_on(domain, compose(g, f), compose(g, f)(point))
        attains_minimum_at(domain, compose(g, f), point)
    }
}

/// Postcomposition by an antitone map turns an attained minimum point into an attained maximum point.
theorem antitone_compose_attains_minimum_to_maximum_at[Domain, Value: PartialOrder, Target: PartialOrder](
    domain: Set[Domain],
    f: Domain -> Value,
    g: Value -> Target,
    point: Domain
) {
    is_antitone(g) and attains_minimum_at(domain, f, point)
    implies attains_maximum_at(domain, compose(g, f), point)
} by {
    if is_antitone(g) and attains_minimum_at(domain, f, point) {
        minimum_point_in_domain(domain, f, point)
        domain.contains(point)
        is_lower_bound_on(domain, f, f(point))
        antitone_compose_lower_bound_to_upper_bound_on(domain, f, g, f(point))
        is_upper_bound_on(domain, compose(g, f), g(f(point)))
        compose(g, f, point) = g(f(point))
        is_upper_bound_on(domain, compose(g, f), compose(g, f)(point))
        attains_maximum_at(domain, compose(g, f), point)
    }
}

/// Postcomposition by a monotone map preserves existence of an attained maximum.
theorem monotone_compose_attains_maximum_on[Domain, Value: PartialOrder, Target: PartialOrder](
    domain: Set[Domain],
    f: Domain -> Value,
    g: Value -> Target
) {
    is_monotone(g) and attains_maximum_on(domain, f)
    implies attains_maximum_on(domain, compose(g, f))
} by {
    if is_monotone(g) and attains_maximum_on(domain, f) {
        let point: Domain satisfy {
            attains_maximum_at(domain, f, point)
        }
        monotone_compose_attains_maximum_at(domain, f, g, point)
        attains_maximum_at(domain, compose(g, f), point)
        exists(witness: Domain) {
            attains_maximum_at(domain, compose(g, f), witness)
        }
        attains_maximum_on(domain, compose(g, f))
    }
}

/// Postcomposition by a monotone map preserves existence of an attained minimum.
theorem monotone_compose_attains_minimum_on[Domain, Value: PartialOrder, Target: PartialOrder](
    domain: Set[Domain],
    f: Domain -> Value,
    g: Value -> Target
) {
    is_monotone(g) and attains_minimum_on(domain, f)
    implies attains_minimum_on(domain, compose(g, f))
} by {
    if is_monotone(g) and attains_minimum_on(domain, f) {
        let point: Domain satisfy {
            attains_minimum_at(domain, f, point)
        }
        monotone_compose_attains_minimum_at(domain, f, g, point)
        attains_minimum_at(domain, compose(g, f), point)
        exists(witness: Domain) {
            attains_minimum_at(domain, compose(g, f), witness)
        }
        attains_minimum_on(domain, compose(g, f))
    }
}

/// Postcomposition by an antitone map turns attained maxima into attained minima.
theorem antitone_compose_attains_maximum_to_minimum_on[Domain, Value: PartialOrder, Target: PartialOrder](
    domain: Set[Domain],
    f: Domain -> Value,
    g: Value -> Target
) {
    is_antitone(g) and attains_maximum_on(domain, f)
    implies attains_minimum_on(domain, compose(g, f))
} by {
    if is_antitone(g) and attains_maximum_on(domain, f) {
        let point: Domain satisfy {
            attains_maximum_at(domain, f, point)
        }
        antitone_compose_attains_maximum_to_minimum_at(domain, f, g, point)
        attains_minimum_at(domain, compose(g, f), point)
        exists(witness: Domain) {
            attains_minimum_at(domain, compose(g, f), witness)
        }
        attains_minimum_on(domain, compose(g, f))
    }
}

/// Postcomposition by an antitone map turns attained minima into attained maxima.
theorem antitone_compose_attains_minimum_to_maximum_on[Domain, Value: PartialOrder, Target: PartialOrder](
    domain: Set[Domain],
    f: Domain -> Value,
    g: Value -> Target
) {
    is_antitone(g) and attains_minimum_on(domain, f)
    implies attains_maximum_on(domain, compose(g, f))
} by {
    if is_antitone(g) and attains_minimum_on(domain, f) {
        let point: Domain satisfy {
            attains_minimum_at(domain, f, point)
        }
        antitone_compose_attains_minimum_to_maximum_at(domain, f, g, point)
        attains_maximum_at(domain, compose(g, f), point)
        exists(witness: Domain) {
            attains_maximum_at(domain, compose(g, f), witness)
        }
        attains_maximum_on(domain, compose(g, f))
    }
}
