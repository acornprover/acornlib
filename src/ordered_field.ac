from algebra.field.field import Field, inverse_inverse
from nat import pow_zero
from algebra.add_ordered_group import AddOrderedGroup
from order import is_monotone, is_antitone, is_strict_monotone, is_strict_antitone, is_order_embedding,
    order_embedding_le_iff_le, order_embedding_lt_iff_lt,
    order_embedding_ge_iff_ge, order_embedding_gt_iff_gt
from order_iso import is_order_iso_pair, is_order_dual_iso_pair, OrderIso, OrderDualIso,
    order_iso_new_map, order_iso_new_inv, order_dual_iso_new_map, order_dual_iso_new_inv,
    order_dual_iso_pair_le_iff_ge, order_dual_iso_pair_lt_iff_gt,
    order_dual_iso_pair_ge_iff_le, order_dual_iso_pair_gt_iff_lt

/// A field with a total order compatible with the field operations.
typeclass F: OrderedField extends Field, AddOrderedGroup {
    /// product of non-negative elements is non-negative
    mul_preserves_nonnegativity(a: F, b: F) {
        F.0 <= a and F.0 <= b implies F.0 <= a * b
    }
}

theorem multiply_inequality_with_nonnegative_element[F: OrderedField](a: F, b: F, c: F) {
    a <= b and F.0 <= c implies a * c <= b * c
} by {
    a + -a <= b + -a
    b + -a = b - a
    a + -a = F.0
    F.0 <= b - a
    F.0 <= (b - a) * c
    (b - a) * c = b * c + -a * c
    a * c <= (b - a) * c + a * c
    a * c <= b * c + (-a * c) + a * c
    (b + -a) * c + a * c = (b + -a + a) * c
    b + -a + a = b + (-a + a)
    -a + a = F.0
    b + F.0 = b
}

theorem multiply_with_nonpositive_flips_inequality[F: OrderedField](a: F, b: F, c: F) {
    a <= b and c <= F.0 implies b * c <= a * c
} by {
    -b <= -a
    F.0 <= -c
    -b * -c <= -a * -c
    -a * -c = a * c
    -b * -c = b * c
}

/// The map obtained by multiplying by a fixed element on the right.
define mul_right_scalar_map[F: OrderedField](c: F, x: F) -> F {
    x * c
}

/// The map obtained by multiplying by a fixed element on the left.
define mul_left_scalar_map[F: OrderedField](c: F, x: F) -> F {
    c * x
}

/// Right multiplication by a nonnegative scalar preserves order.
theorem mul_right_scalar_map_is_monotone_of_nonnegative[F: OrderedField](c: F) {
    F.0 <= c implies is_monotone(mul_right_scalar_map(c))
} by {
    if F.0 <= c {
        forall(x: F, y: F) {
            if x <= y {
                multiply_inequality_with_nonnegative_element(x, y, c)
                x * c <= y * c
                mul_right_scalar_map(c, x) <= mul_right_scalar_map(c, y)
            }
        }
    }
}

/// Left multiplication by a nonnegative scalar preserves order.
theorem mul_left_scalar_map_is_monotone_of_nonnegative[F: OrderedField](c: F) {
    F.0 <= c implies is_monotone(mul_left_scalar_map(c))
} by {
    if F.0 <= c {
        forall(x: F, y: F) {
            if x <= y {
                multiply_inequality_with_nonnegative_element(x, y, c)
                x * c <= y * c
                c * x = x * c
                c * y = y * c
                mul_left_scalar_map(c, x) <= mul_left_scalar_map(c, y)
            }
        }
    }
}

/// Right multiplication by a nonpositive scalar reverses order.
theorem mul_right_scalar_map_is_antitone_of_nonpositive[F: OrderedField](c: F) {
    c <= F.0 implies is_antitone(mul_right_scalar_map(c))
} by {
    if c <= F.0 {
        forall(x: F, y: F) {
            if x <= y {
                multiply_with_nonpositive_flips_inequality(x, y, c)
                y * c <= x * c
                mul_right_scalar_map(c, y) <= mul_right_scalar_map(c, x)
            }
        }
    }
}

/// Left multiplication by a nonpositive scalar reverses order.
theorem mul_left_scalar_map_is_antitone_of_nonpositive[F: OrderedField](c: F) {
    c <= F.0 implies is_antitone(mul_left_scalar_map(c))
} by {
    if c <= F.0 {
        forall(x: F, y: F) {
            if x <= y {
                multiply_with_nonpositive_flips_inequality(x, y, c)
                y * c <= x * c
                c * y = y * c
                c * x = x * c
                mul_left_scalar_map(c, y) <= mul_left_scalar_map(c, x)
            }
        }
    }
}

/// Right multiplication by a positive scalar reflects the non-strict order.
theorem mul_right_scalar_map_reflects_le_of_positive[F: OrderedField](c: F, x: F, y: F) {
    F.0 < c and F.0 <= c.inverse and mul_right_scalar_map(c, x) <= mul_right_scalar_map(c, y) implies x <= y
} by {
    if F.0 < c and F.0 <= c.inverse and mul_right_scalar_map(c, x) <= mul_right_scalar_map(c, y) {
        mul_right_scalar_map(c, x) = x * c
        mul_right_scalar_map(c, y) = y * c
        multiply_inequality_with_nonnegative_element(x * c, y * c, c.inverse)
        (x * c) * c.inverse <= (y * c) * c.inverse
        (x * c) * c.inverse = x * (c * c.inverse)
        (y * c) * c.inverse = y * (c * c.inverse)
        c != F.0
        c * c.inverse = F.1
        x * F.1 = x
        y * F.1 = y
        x <= y
    }
}

/// Left multiplication by a positive scalar reflects the non-strict order.
theorem mul_left_scalar_map_reflects_le_of_positive[F: OrderedField](c: F, x: F, y: F) {
    F.0 < c and F.0 <= c.inverse and mul_left_scalar_map(c, x) <= mul_left_scalar_map(c, y) implies x <= y
} by {
    if F.0 < c and F.0 <= c.inverse and mul_left_scalar_map(c, x) <= mul_left_scalar_map(c, y) {
        mul_left_scalar_map(c, x) = c * x
        mul_left_scalar_map(c, y) = c * y
        c * x = x * c
        c * y = y * c
        mul_right_scalar_map_reflects_le_of_positive(c, x, y)
        x <= y
    }
}

/// Right multiplication by a positive scalar is an order embedding.
theorem mul_right_scalar_map_is_order_embedding_of_positive[F: OrderedField](c: F) {
    F.0 < c and F.0 <= c.inverse implies is_order_embedding(mul_right_scalar_map(c))
} by {
    if F.0 < c and F.0 <= c.inverse {
        forall(x: F, y: F) {
            if mul_right_scalar_map(c, x) <= mul_right_scalar_map(c, y) {
                mul_right_scalar_map_reflects_le_of_positive(c, x, y)
                x <= y
            }
            if x <= y {
                F.0 <= c
                mul_right_scalar_map_is_monotone_of_nonnegative(c)
                is_monotone(mul_right_scalar_map(c))
                mul_right_scalar_map(c, x) <= mul_right_scalar_map(c, y)
            }
            mul_right_scalar_map(c, x) <= mul_right_scalar_map(c, y) = (x <= y)
        }
    }
}

/// Left multiplication by a positive scalar is an order embedding.
theorem mul_left_scalar_map_is_order_embedding_of_positive[F: OrderedField](c: F) {
    F.0 < c and F.0 <= c.inverse implies is_order_embedding(mul_left_scalar_map(c))
} by {
    if F.0 < c and F.0 <= c.inverse {
        forall(x: F, y: F) {
            if mul_left_scalar_map(c, x) <= mul_left_scalar_map(c, y) {
                mul_left_scalar_map_reflects_le_of_positive(c, x, y)
                x <= y
            }
            if x <= y {
                F.0 <= c
                mul_left_scalar_map_is_monotone_of_nonnegative(c)
                is_monotone(mul_left_scalar_map(c))
                mul_left_scalar_map(c, x) <= mul_left_scalar_map(c, y)
            }
            mul_left_scalar_map(c, x) <= mul_left_scalar_map(c, y) = (x <= y)
        }
    }
}

/// Right multiplication by the inverse is a left inverse of right multiplication.
theorem mul_right_scalar_map_left_inverse[F: OrderedField](c: F, x: F) {
    c != F.0 implies mul_right_scalar_map(c.inverse, mul_right_scalar_map(c, x)) = x
} by {
    if c != F.0 {
        mul_right_scalar_map(c.inverse, mul_right_scalar_map(c, x)) = mul_right_scalar_map(c, x) * c.inverse
        mul_right_scalar_map(c, x) = x * c
        (x * c) * c.inverse = x * (c * c.inverse)
        c * c.inverse = F.1
        x * F.1 = x
        mul_right_scalar_map(c.inverse, mul_right_scalar_map(c, x)) = x
    }
}

/// Right multiplication is a left inverse of right multiplication by the inverse.
theorem mul_right_scalar_map_right_inverse[F: OrderedField](c: F, x: F) {
    c != F.0 implies mul_right_scalar_map(c, mul_right_scalar_map(c.inverse, x)) = x
} by {
    if c != F.0 {
        mul_right_scalar_map(c, mul_right_scalar_map(c.inverse, x)) = mul_right_scalar_map(c.inverse, x) * c
        mul_right_scalar_map(c.inverse, x) = x * c.inverse
        (x * c.inverse) * c = x * (c.inverse * c)
        c.inverse * c = c * c.inverse
        c * c.inverse = F.1
        x * F.1 = x
        mul_right_scalar_map(c, mul_right_scalar_map(c.inverse, x)) = x
    }
}

/// Left multiplication by the inverse is a left inverse of left multiplication.
theorem mul_left_scalar_map_left_inverse[F: OrderedField](c: F, x: F) {
    c != F.0 implies mul_left_scalar_map(c.inverse, mul_left_scalar_map(c, x)) = x
} by {
    if c != F.0 {
        mul_left_scalar_map(c.inverse, mul_left_scalar_map(c, x)) = c.inverse * mul_left_scalar_map(c, x)
        mul_left_scalar_map(c, x) = c * x
        c.inverse * (c * x) = c.inverse * c * x
        c.inverse * c = c * c.inverse
        c * c.inverse = F.1
        F.1 * x = x
        mul_left_scalar_map(c.inverse, mul_left_scalar_map(c, x)) = x
    }
}

/// Left multiplication is a left inverse of left multiplication by the inverse.
theorem mul_left_scalar_map_right_inverse[F: OrderedField](c: F, x: F) {
    c != F.0 implies mul_left_scalar_map(c, mul_left_scalar_map(c.inverse, x)) = x
} by {
    if c != F.0 {
        mul_left_scalar_map(c, mul_left_scalar_map(c.inverse, x)) = c * mul_left_scalar_map(c.inverse, x)
        mul_left_scalar_map(c.inverse, x) = c.inverse * x
        c * (c.inverse * x) = c * c.inverse * x
        c * c.inverse = F.1
        F.1 * x = x
        mul_left_scalar_map(c, mul_left_scalar_map(c.inverse, x)) = x
    }
}

/// Right multiplication by a positive scalar and its inverse form an order isomorphism pair.
theorem mul_right_scalar_map_is_order_iso_pair_of_positive[F: OrderedField](c: F) {
    F.0 < c and F.0 < c.inverse implies is_order_iso_pair(mul_right_scalar_map(c), mul_right_scalar_map(c.inverse))
} by {
    if F.0 < c and F.0 < c.inverse {
        let f = mul_right_scalar_map(c)
        let g = mul_right_scalar_map(c.inverse)
        c != F.0
        F.0 <= c.inverse
        c.inverse.inverse = c
        F.0 <= c.inverse.inverse
        let left_inverse: Bool = forall(x: F) {
            g(f(x)) = x
        }
        let right_inverse: Bool = forall(y: F) {
            f(g(y)) = y
        }
        forall(x: F) {
            mul_right_scalar_map_left_inverse(c, x)
            g(f(x)) = x
        }
        left_inverse
        forall(y: F) {
            mul_right_scalar_map_right_inverse(c, y)
            f(g(y)) = y
        }
        right_inverse
        mul_right_scalar_map_is_order_embedding_of_positive(c)
        is_order_embedding(f)
        mul_right_scalar_map_is_order_embedding_of_positive(c.inverse)
        is_order_embedding(g)
        left_inverse and right_inverse
        left_inverse and right_inverse and is_order_embedding(f)
        left_inverse and right_inverse and is_order_embedding(f) and is_order_embedding(g)
        is_order_iso_pair(f, g) = (left_inverse and right_inverse and is_order_embedding(f) and is_order_embedding(g))
        left_inverse = forall(x: F) {
            g(f(x)) = x
        }
        right_inverse = forall(y: F) {
            f(g(y)) = y
        }
        is_order_iso_pair(f, g)
        f = mul_right_scalar_map(c)
        g = mul_right_scalar_map(c.inverse)
        is_order_iso_pair(mul_right_scalar_map(c), mul_right_scalar_map(c.inverse))
    }
}

/// Left multiplication by a positive scalar and its inverse form an order isomorphism pair.
theorem mul_left_scalar_map_is_order_iso_pair_of_positive[F: OrderedField](c: F) {
    F.0 < c and F.0 < c.inverse implies is_order_iso_pair(mul_left_scalar_map(c), mul_left_scalar_map(c.inverse))
} by {
    if F.0 < c and F.0 < c.inverse {
        let f = mul_left_scalar_map(c)
        let g = mul_left_scalar_map(c.inverse)
        c != F.0
        F.0 <= c.inverse
        c.inverse.inverse = c
        F.0 <= c.inverse.inverse
        let left_inverse: Bool = forall(x: F) {
            g(f(x)) = x
        }
        let right_inverse: Bool = forall(y: F) {
            f(g(y)) = y
        }
        forall(x: F) {
            mul_left_scalar_map_left_inverse(c, x)
            g(f(x)) = x
        }
        left_inverse
        forall(y: F) {
            mul_left_scalar_map_right_inverse(c, y)
            f(g(y)) = y
        }
        right_inverse
        mul_left_scalar_map_is_order_embedding_of_positive(c)
        is_order_embedding(f)
        mul_left_scalar_map_is_order_embedding_of_positive(c.inverse)
        is_order_embedding(g)
        left_inverse and right_inverse
        left_inverse and right_inverse and is_order_embedding(f)
        left_inverse and right_inverse and is_order_embedding(f) and is_order_embedding(g)
        is_order_iso_pair(f, g) = (left_inverse and right_inverse and is_order_embedding(f) and is_order_embedding(g))
        left_inverse = forall(x: F) {
            g(f(x)) = x
        }
        right_inverse = forall(y: F) {
            f(g(y)) = y
        }
        is_order_iso_pair(f, g)
        f = mul_left_scalar_map(c)
        g = mul_left_scalar_map(c.inverse)
        is_order_iso_pair(mul_left_scalar_map(c), mul_left_scalar_map(c.inverse))
    }
}

/// Right multiplication by a positive scalar preserves and reflects non-strict order.
theorem mul_right_scalar_map_le_iff_le_of_positive[F: OrderedField](c: F, x: F, y: F) {
    F.0 < c and F.0 <= c.inverse implies (mul_right_scalar_map(c, x) <= mul_right_scalar_map(c, y) = (x <= y))
} by {
    if F.0 < c and F.0 <= c.inverse {
        mul_right_scalar_map_is_order_embedding_of_positive(c)
        order_embedding_le_iff_le(mul_right_scalar_map(c), x, y)
    }
}

/// Left multiplication by a positive scalar preserves and reflects non-strict order.
theorem mul_left_scalar_map_le_iff_le_of_positive[F: OrderedField](c: F, x: F, y: F) {
    F.0 < c and F.0 <= c.inverse implies (mul_left_scalar_map(c, x) <= mul_left_scalar_map(c, y) = (x <= y))
} by {
    if F.0 < c and F.0 <= c.inverse {
        mul_left_scalar_map_is_order_embedding_of_positive(c)
        order_embedding_le_iff_le(mul_left_scalar_map(c), x, y)
    }
}

/// Right multiplication by a positive scalar preserves and reflects strict order.
theorem mul_right_scalar_map_lt_iff_lt_of_positive[F: OrderedField](c: F, x: F, y: F) {
    F.0 < c and F.0 <= c.inverse implies (mul_right_scalar_map(c, x) < mul_right_scalar_map(c, y) = (x < y))
} by {
    if F.0 < c and F.0 <= c.inverse {
        mul_right_scalar_map_is_order_embedding_of_positive(c)
        order_embedding_lt_iff_lt(mul_right_scalar_map(c), x, y)
    }
}

/// Left multiplication by a positive scalar preserves and reflects strict order.
theorem mul_left_scalar_map_lt_iff_lt_of_positive[F: OrderedField](c: F, x: F, y: F) {
    F.0 < c and F.0 <= c.inverse implies (mul_left_scalar_map(c, x) < mul_left_scalar_map(c, y) = (x < y))
} by {
    if F.0 < c and F.0 <= c.inverse {
        mul_left_scalar_map_is_order_embedding_of_positive(c)
        order_embedding_lt_iff_lt(mul_left_scalar_map(c), x, y)
    }
}

/// Right multiplication by a positive scalar preserves and reflects reverse non-strict order.
theorem mul_right_scalar_map_ge_iff_ge_of_positive[F: OrderedField](c: F, x: F, y: F) {
    F.0 < c and F.0 <= c.inverse implies (mul_right_scalar_map(c, x) >= mul_right_scalar_map(c, y) = (x >= y))
} by {
    if F.0 < c and F.0 <= c.inverse {
        mul_right_scalar_map_is_order_embedding_of_positive(c)
        order_embedding_ge_iff_ge(mul_right_scalar_map(c), x, y)
    }
}

/// Left multiplication by a positive scalar preserves and reflects reverse non-strict order.
theorem mul_left_scalar_map_ge_iff_ge_of_positive[F: OrderedField](c: F, x: F, y: F) {
    F.0 < c and F.0 <= c.inverse implies (mul_left_scalar_map(c, x) >= mul_left_scalar_map(c, y) = (x >= y))
} by {
    if F.0 < c and F.0 <= c.inverse {
        mul_left_scalar_map_is_order_embedding_of_positive(c)
        order_embedding_ge_iff_ge(mul_left_scalar_map(c), x, y)
    }
}

/// Right multiplication by a positive scalar preserves and reflects reverse strict order.
theorem mul_right_scalar_map_gt_iff_gt_of_positive[F: OrderedField](c: F, x: F, y: F) {
    F.0 < c and F.0 <= c.inverse implies (mul_right_scalar_map(c, x) > mul_right_scalar_map(c, y) = (x > y))
} by {
    if F.0 < c and F.0 <= c.inverse {
        mul_right_scalar_map_is_order_embedding_of_positive(c)
        order_embedding_gt_iff_gt(mul_right_scalar_map(c), x, y)
    }
}

/// Left multiplication by a positive scalar preserves and reflects reverse strict order.
theorem mul_left_scalar_map_gt_iff_gt_of_positive[F: OrderedField](c: F, x: F, y: F) {
    F.0 < c and F.0 <= c.inverse implies (mul_left_scalar_map(c, x) > mul_left_scalar_map(c, y) = (x > y))
} by {
    if F.0 < c and F.0 <= c.inverse {
        mul_left_scalar_map_is_order_embedding_of_positive(c)
        order_embedding_gt_iff_gt(mul_left_scalar_map(c), x, y)
    }
}

/// Right multiplication by a negative scalar and its inverse form an order-dual isomorphism pair.
theorem mul_right_scalar_map_is_order_dual_iso_pair_of_negative[F: OrderedField](c: F) {
    c < F.0 and c.inverse < F.0 implies is_order_dual_iso_pair(mul_right_scalar_map(c), mul_right_scalar_map(c.inverse))
} by {
    if c < F.0 and c.inverse < F.0 {
        let f = mul_right_scalar_map(c)
        let g = mul_right_scalar_map(c.inverse)
        c != F.0
        c <= F.0
        c.inverse <= F.0
        c.inverse.inverse = c
        c.inverse.inverse <= F.0
        let left_inverse: Bool = forall(x: F) {
            g(f(x)) = x
        }
        let right_inverse: Bool = forall(y: F) {
            f(g(y)) = y
        }
        forall(x: F) {
            mul_right_scalar_map_left_inverse(c, x)
            g(f(x)) = x
        }
        left_inverse
        forall(y: F) {
            mul_right_scalar_map_right_inverse(c, y)
            f(g(y)) = y
        }
        right_inverse
        mul_right_scalar_map_is_antitone_of_nonpositive(c)
        is_antitone(f)
        mul_right_scalar_map_is_antitone_of_nonpositive(c.inverse)
        is_antitone(g)
        left_inverse and right_inverse
        left_inverse and right_inverse and is_antitone(f)
        left_inverse and right_inverse and is_antitone(f) and is_antitone(g)
        is_order_dual_iso_pair(f, g) = (left_inverse and right_inverse and is_antitone(f) and is_antitone(g))
        left_inverse = forall(x: F) {
            g(f(x)) = x
        }
        right_inverse = forall(y: F) {
            f(g(y)) = y
        }
        is_order_dual_iso_pair(f, g)
        f = mul_right_scalar_map(c)
        g = mul_right_scalar_map(c.inverse)
        is_order_dual_iso_pair(mul_right_scalar_map(c), mul_right_scalar_map(c.inverse))
    }
}

/// Left multiplication by a negative scalar and its inverse form an order-dual isomorphism pair.
theorem mul_left_scalar_map_is_order_dual_iso_pair_of_negative[F: OrderedField](c: F) {
    c < F.0 and c.inverse < F.0 implies is_order_dual_iso_pair(mul_left_scalar_map(c), mul_left_scalar_map(c.inverse))
} by {
    if c < F.0 and c.inverse < F.0 {
        let f = mul_left_scalar_map(c)
        let g = mul_left_scalar_map(c.inverse)
        c != F.0
        c <= F.0
        c.inverse <= F.0
        c.inverse.inverse = c
        c.inverse.inverse <= F.0
        let left_inverse: Bool = forall(x: F) {
            g(f(x)) = x
        }
        let right_inverse: Bool = forall(y: F) {
            f(g(y)) = y
        }
        forall(x: F) {
            mul_left_scalar_map_left_inverse(c, x)
            g(f(x)) = x
        }
        left_inverse
        forall(y: F) {
            mul_left_scalar_map_right_inverse(c, y)
            f(g(y)) = y
        }
        right_inverse
        mul_left_scalar_map_is_antitone_of_nonpositive(c)
        is_antitone(f)
        mul_left_scalar_map_is_antitone_of_nonpositive(c.inverse)
        is_antitone(g)
        left_inverse and right_inverse
        left_inverse and right_inverse and is_antitone(f)
        left_inverse and right_inverse and is_antitone(f) and is_antitone(g)
        is_order_dual_iso_pair(f, g) = (left_inverse and right_inverse and is_antitone(f) and is_antitone(g))
        left_inverse = forall(x: F) {
            g(f(x)) = x
        }
        right_inverse = forall(y: F) {
            f(g(y)) = y
        }
        is_order_dual_iso_pair(f, g)
        f = mul_left_scalar_map(c)
        g = mul_left_scalar_map(c.inverse)
        is_order_dual_iso_pair(mul_left_scalar_map(c), mul_left_scalar_map(c.inverse))
    }
}

/// Right multiplication by a negative scalar reverses and reflects non-strict order.
theorem mul_right_scalar_map_le_iff_ge_of_negative[F: OrderedField](c: F, x: F, y: F) {
    c < F.0 and c.inverse < F.0 implies (mul_right_scalar_map(c, x) <= mul_right_scalar_map(c, y) = (x >= y))
} by {
    if c < F.0 and c.inverse < F.0 {
        mul_right_scalar_map_is_order_dual_iso_pair_of_negative(c)
        order_dual_iso_pair_le_iff_ge(mul_right_scalar_map(c), mul_right_scalar_map(c.inverse), x, y)
    }
}

/// Left multiplication by a negative scalar reverses and reflects non-strict order.
theorem mul_left_scalar_map_le_iff_ge_of_negative[F: OrderedField](c: F, x: F, y: F) {
    c < F.0 and c.inverse < F.0 implies (mul_left_scalar_map(c, x) <= mul_left_scalar_map(c, y) = (x >= y))
} by {
    if c < F.0 and c.inverse < F.0 {
        mul_left_scalar_map_is_order_dual_iso_pair_of_negative(c)
        order_dual_iso_pair_le_iff_ge(mul_left_scalar_map(c), mul_left_scalar_map(c.inverse), x, y)
    }
}

/// Right multiplication by a negative scalar reverses and reflects strict order.
theorem mul_right_scalar_map_lt_iff_gt_of_negative[F: OrderedField](c: F, x: F, y: F) {
    c < F.0 and c.inverse < F.0 implies (mul_right_scalar_map(c, x) < mul_right_scalar_map(c, y) = (x > y))
} by {
    if c < F.0 and c.inverse < F.0 {
        mul_right_scalar_map_is_order_dual_iso_pair_of_negative(c)
        order_dual_iso_pair_lt_iff_gt(mul_right_scalar_map(c), mul_right_scalar_map(c.inverse), x, y)
    }
}

/// Left multiplication by a negative scalar reverses and reflects strict order.
theorem mul_left_scalar_map_lt_iff_gt_of_negative[F: OrderedField](c: F, x: F, y: F) {
    c < F.0 and c.inverse < F.0 implies (mul_left_scalar_map(c, x) < mul_left_scalar_map(c, y) = (x > y))
} by {
    if c < F.0 and c.inverse < F.0 {
        mul_left_scalar_map_is_order_dual_iso_pair_of_negative(c)
        order_dual_iso_pair_lt_iff_gt(mul_left_scalar_map(c), mul_left_scalar_map(c.inverse), x, y)
    }
}

/// Right multiplication by a negative scalar reverses and reflects reverse non-strict order.
theorem mul_right_scalar_map_ge_iff_le_of_negative[F: OrderedField](c: F, x: F, y: F) {
    c < F.0 and c.inverse < F.0 implies (mul_right_scalar_map(c, x) >= mul_right_scalar_map(c, y) = (x <= y))
} by {
    if c < F.0 and c.inverse < F.0 {
        mul_right_scalar_map_is_order_dual_iso_pair_of_negative(c)
        order_dual_iso_pair_ge_iff_le(mul_right_scalar_map(c), mul_right_scalar_map(c.inverse), x, y)
    }
}

/// Left multiplication by a negative scalar reverses and reflects reverse non-strict order.
theorem mul_left_scalar_map_ge_iff_le_of_negative[F: OrderedField](c: F, x: F, y: F) {
    c < F.0 and c.inverse < F.0 implies (mul_left_scalar_map(c, x) >= mul_left_scalar_map(c, y) = (x <= y))
} by {
    if c < F.0 and c.inverse < F.0 {
        mul_left_scalar_map_is_order_dual_iso_pair_of_negative(c)
        order_dual_iso_pair_ge_iff_le(mul_left_scalar_map(c), mul_left_scalar_map(c.inverse), x, y)
    }
}

/// Right multiplication by a negative scalar reverses and reflects strict reverse order.
theorem mul_right_scalar_map_gt_iff_lt_of_negative[F: OrderedField](c: F, x: F, y: F) {
    c < F.0 and c.inverse < F.0 implies (mul_right_scalar_map(c, x) > mul_right_scalar_map(c, y) = (x < y))
} by {
    if c < F.0 and c.inverse < F.0 {
        mul_right_scalar_map_is_order_dual_iso_pair_of_negative(c)
        order_dual_iso_pair_gt_iff_lt(mul_right_scalar_map(c), mul_right_scalar_map(c.inverse), x, y)
    }
}

/// Left multiplication by a negative scalar reverses and reflects strict reverse order.
theorem mul_left_scalar_map_gt_iff_lt_of_negative[F: OrderedField](c: F, x: F, y: F) {
    c < F.0 and c.inverse < F.0 implies (mul_left_scalar_map(c, x) > mul_left_scalar_map(c, y) = (x < y))
} by {
    if c < F.0 and c.inverse < F.0 {
        mul_left_scalar_map_is_order_dual_iso_pair_of_negative(c)
        order_dual_iso_pair_gt_iff_lt(mul_left_scalar_map(c), mul_left_scalar_map(c.inverse), x, y)
    }
}

/// Multiplication on the right by a nonnegative scalar preserves order.
theorem mul_le_mul_of_nonneg_right[F: OrderedField](a: F, b: F, c: F) {
    a <= b and F.0 <= c implies a * c <= b * c
} by {
    if a <= b and F.0 <= c {
        multiply_inequality_with_nonnegative_element(a, b, c)
        a * c <= b * c
    }
}

/// Multiplication on the left by a nonnegative scalar preserves order.
theorem mul_le_mul_of_nonneg_left[F: OrderedField](a: F, b: F, c: F) {
    a <= b and F.0 <= c implies c * a <= c * b
} by {
    if a <= b and F.0 <= c {
        multiply_inequality_with_nonnegative_element(a, b, c)
        a * c <= b * c
        c * a = a * c
        c * b = b * c
        c * a <= c * b
    }
}

/// Multiplication on the right by a nonpositive scalar reverses order.
theorem mul_le_mul_of_nonpos_right[F: OrderedField](a: F, b: F, c: F) {
    a <= b and c <= F.0 implies b * c <= a * c
} by {
    if a <= b and c <= F.0 {
        multiply_with_nonpositive_flips_inequality(a, b, c)
        b * c <= a * c
    }
}

/// Multiplication on the left by a nonpositive scalar reverses order.
theorem mul_le_mul_of_nonpos_left[F: OrderedField](a: F, b: F, c: F) {
    a <= b and c <= F.0 implies c * b <= c * a
} by {
    if a <= b and c <= F.0 {
        multiply_with_nonpositive_flips_inequality(a, b, c)
        b * c <= a * c
        c * b = b * c
        c * a = a * c
        c * b <= c * a
    }
}

/// Multiplication on the right by a positive scalar preserves strict order.
theorem mul_lt_mul_of_pos_right[F: OrderedField](a: F, b: F, c: F) {
    a < b and F.0 < c implies a * c < b * c
} by {
    if a < b and F.0 < c {
        a <= b
        F.0 <= c
        mul_le_mul_of_nonneg_right(a, b, c)
        a * c <= b * c
        if a * c = b * c {
            c != F.0
            a * c * c.inverse = b * c * c.inverse
            a * (c * c.inverse) = b * (c * c.inverse)
            c * c.inverse = F.1
            a * F.1 = a
            b * F.1 = b
            a = b
            a != b
            false
        }
        a * c != b * c
        a * c < b * c
    }
}

/// Product of positive ordered-field elements is positive.
theorem mul_pos_pos[F: OrderedField](a: F, b: F) {
    F.0 < a and F.0 < b implies F.0 < a * b
} by {
    if F.0 < a and F.0 < b {
        mul_lt_mul_of_pos_right(F.0, a, b)
        F.0 * b = F.0
        F.0 < a * b
    }
}

/// Multiplication on the left by a positive scalar preserves strict order.
theorem mul_lt_mul_of_pos_left[F: OrderedField](a: F, b: F, c: F) {
    a < b and F.0 < c implies c * a < c * b
} by {
    if a < b and F.0 < c {
        mul_lt_mul_of_pos_right(a, b, c)
        a * c < b * c
        c * a = a * c
        c * b = b * c
        c * a < c * b
    }
}

/// Multiplication on the right by a negative scalar reverses strict order.
theorem mul_lt_mul_of_neg_right[F: OrderedField](a: F, b: F, c: F) {
    a < b and c < F.0 implies b * c < a * c
} by {
    if a < b and c < F.0 {
        a <= b
        c <= F.0
        mul_le_mul_of_nonpos_right(a, b, c)
        b * c <= a * c
        if b * c = a * c {
            c != F.0
            b * c * c.inverse = a * c * c.inverse
            b * (c * c.inverse) = a * (c * c.inverse)
            c * c.inverse = F.1
            b * F.1 = b
            a * F.1 = a
            b = a
            a = b
            a != b
            false
        }
        b * c != a * c
        b * c < a * c
    }
}

/// Multiplication on the left by a negative scalar reverses strict order.
theorem mul_lt_mul_of_neg_left[F: OrderedField](a: F, b: F, c: F) {
    a < b and c < F.0 implies c * b < c * a
} by {
    if a < b and c < F.0 {
        mul_lt_mul_of_neg_right(a, b, c)
        b * c < a * c
        c * b = b * c
        c * a = a * c
        c * b < c * a
    }
}

/// Multiplication on the right by a nonnegative scalar preserves reverse order.
theorem mul_ge_mul_of_nonneg_right[F: OrderedField](a: F, b: F, c: F) {
    a >= b and F.0 <= c implies a * c >= b * c
} by {
    if a >= b and F.0 <= c {
        b <= a
        mul_le_mul_of_nonneg_right(b, a, c)
        b * c <= a * c
        a * c >= b * c
    }
}

/// Multiplication on the left by a nonnegative scalar preserves reverse order.
theorem mul_ge_mul_of_nonneg_left[F: OrderedField](a: F, b: F, c: F) {
    a >= b and F.0 <= c implies c * a >= c * b
} by {
    if a >= b and F.0 <= c {
        b <= a
        mul_le_mul_of_nonneg_left(b, a, c)
        c * b <= c * a
        c * a >= c * b
    }
}

/// Multiplication on the right by a nonpositive scalar reverses reverse order.
theorem mul_ge_mul_of_nonpos_right[F: OrderedField](a: F, b: F, c: F) {
    a >= b and c <= F.0 implies b * c >= a * c
} by {
    if a >= b and c <= F.0 {
        b <= a
        mul_le_mul_of_nonpos_right(b, a, c)
        a * c <= b * c
        b * c >= a * c
    }
}

/// Multiplication on the left by a nonpositive scalar reverses reverse order.
theorem mul_ge_mul_of_nonpos_left[F: OrderedField](a: F, b: F, c: F) {
    a >= b and c <= F.0 implies c * b >= c * a
} by {
    if a >= b and c <= F.0 {
        b <= a
        mul_le_mul_of_nonpos_left(b, a, c)
        c * a <= c * b
        c * b >= c * a
    }
}

/// Multiplication on the right by a positive scalar preserves strict reverse order.
theorem mul_gt_mul_of_pos_right[F: OrderedField](a: F, b: F, c: F) {
    a > b and F.0 < c implies a * c > b * c
} by {
    if a > b and F.0 < c {
        b < a
        mul_lt_mul_of_pos_right(b, a, c)
        b * c < a * c
        a * c > b * c
    }
}

/// Multiplication on the left by a positive scalar preserves strict reverse order.
theorem mul_gt_mul_of_pos_left[F: OrderedField](a: F, b: F, c: F) {
    a > b and F.0 < c implies c * a > c * b
} by {
    if a > b and F.0 < c {
        b < a
        mul_lt_mul_of_pos_left(b, a, c)
        c * b < c * a
        c * a > c * b
    }
}

/// Multiplication on the right by a negative scalar reverses strict reverse order.
theorem mul_gt_mul_of_neg_right[F: OrderedField](a: F, b: F, c: F) {
    a > b and c < F.0 implies b * c > a * c
} by {
    if a > b and c < F.0 {
        b < a
        mul_lt_mul_of_neg_right(b, a, c)
        a * c < b * c
        b * c > a * c
    }
}

/// Multiplication on the left by a negative scalar reverses strict reverse order.
theorem mul_gt_mul_of_neg_left[F: OrderedField](a: F, b: F, c: F) {
    a > b and c < F.0 implies c * b > c * a
} by {
    if a > b and c < F.0 {
        b < a
        mul_lt_mul_of_neg_left(b, a, c)
        c * a < c * b
        c * b > c * a
    }
}

/// Right multiplication by a positive scalar preserves strict order.
theorem mul_right_scalar_map_is_strict_monotone_of_positive[F: OrderedField](c: F) {
    F.0 < c implies is_strict_monotone(mul_right_scalar_map(c))
} by {
    if F.0 < c {
        forall(x: F, y: F) {
            if x < y {
                mul_lt_mul_of_pos_right(x, y, c)
                x * c < y * c
                mul_right_scalar_map(c, x) < mul_right_scalar_map(c, y)
            }
        }
    }
}

/// Left multiplication by a positive scalar preserves strict order.
theorem mul_left_scalar_map_is_strict_monotone_of_positive[F: OrderedField](c: F) {
    F.0 < c implies is_strict_monotone(mul_left_scalar_map(c))
} by {
    if F.0 < c {
        forall(x: F, y: F) {
            if x < y {
                mul_lt_mul_of_pos_left(x, y, c)
                c * x < c * y
                mul_left_scalar_map(c, x) < mul_left_scalar_map(c, y)
            }
        }
    }
}

/// Right multiplication by a negative scalar reverses strict order.
theorem mul_right_scalar_map_is_strict_antitone_of_negative[F: OrderedField](c: F) {
    c < F.0 implies is_strict_antitone(mul_right_scalar_map(c))
} by {
    if c < F.0 {
        forall(x: F, y: F) {
            if x < y {
                mul_lt_mul_of_neg_right(x, y, c)
                y * c < x * c
                mul_right_scalar_map(c, y) < mul_right_scalar_map(c, x)
            }
        }
    }
}

/// Left multiplication by a negative scalar reverses strict order.
theorem mul_left_scalar_map_is_strict_antitone_of_negative[F: OrderedField](c: F) {
    c < F.0 implies is_strict_antitone(mul_left_scalar_map(c))
} by {
    if c < F.0 {
        forall(x: F, y: F) {
            if x < y {
                mul_lt_mul_of_neg_left(x, y, c)
                c * y < c * x
                mul_left_scalar_map(c, y) < mul_left_scalar_map(c, x)
            }
        }
    }
}

theorem inverse_on_positive_flips_inequality[F: OrderedField](a: F, b: F) {
    a < b and F.0 < a and F.0 < b implies b.inverse < a.inverse
} by {
    F.0 <= a
    F.0 != a
    a * a.inverse = F.1
    a.inverse * a = a * a.inverse
    a.inverse <= F.0 or F.0 <= a.inverse
    F.0 * a = F.0
    F.0 <= a.inverse
    F.0 <= b
    F.0 != b
    b * b.inverse = F.1
    b.inverse * b = b * b.inverse
    b.inverse <= F.0 or F.0 <= b.inverse
    F.0 * b = F.0
    F.0 <= b.inverse
    F.1 <= b * a.inverse
    b.inverse * b = F.1
    F.1 * b.inverse <= b * a.inverse * b.inverse
    b * a.inverse * b.inverse = b * (a.inverse * b.inverse)
    b.inverse * a.inverse = a.inverse * b.inverse
    b * b.inverse * a.inverse = b * (b.inverse * a.inverse)
    F.1 * b.inverse = b.inverse
    b.inverse <= F.1 * a.inverse
}

theorem squares_are_nonnegative[F: OrderedField](a: F) {
    a * a >= F.0
} by {
    if a >= F.0 {

    } else {
    }
}

from nat import Nat

theorem even_pows_are_nonnegative[F: OrderedField](a: F, n: Nat) {
    a.pow(Nat.2 * n) >= F.0
} by {
    a.pow(n) * a.pow(n) = a.pow(n + n)
    a.pow(n) * a.pow(n) >= F.0
    Nat.1 * n = n
    n + Nat.1 * n = Nat.1.suc * n
}

from int import Int

theorem nonnegative_even_zpows_are_nonnegative[F: OrderedField](a: F, n: Int) {
    n >= Int.0 implies a.zpow(Int.2 * n) >= F.0
} by {
    a.zpow(n) * a.zpow(n) = a.zpow(n + n)
    Int.2 * n = n + n
    a.zpow(n) * a.zpow(n) >= F.0
}

theorem even_zpows_are_nonnegative[F: OrderedField](a: F, n: Int) {
    a.zpow(Int.2 * n) >= F.0
} by {
    if n >= Int.0 {
    } else {
        n <= Int.0
        a.zpow(Int.2 * -n).inverse >= F.0
    }
}

theorem zero_is_smaller_than_one[F: OrderedField] {
    F.0 < F.1
}

theorem pows_of_nonnegative_are_nonnegative[F: OrderedField](a: F, n: Nat) {
    F.0 <= a implies F.0 <= a.pow(n)
} by {
    // Define a helper function for induction
    define f(x: Nat) -> Bool { F.0 <= a.pow(x) }

    // Base case
    f(Nat.0)

    // Inductive step
    forall(k: Nat) {
        a * a.pow(k) = a.pow(k.suc)
        f(k) implies f(k.suc)
    }
}

theorem nonnegative_zpows_of_nonnegative_are_nonnegative[F: OrderedField](a: F, n: Int) {
    F.0 <= a and n >= Int.0 implies F.0 <= a.zpow(n)
} by {
    let k: Nat satisfy {
        n = Int.from_nat(k)
    }
}

theorem inverse_of_nonnegative_is_nonnegative[F: OrderedField](a: F) {
    F.0 <= a implies F.0 <= a.inverse
} by {
    if F.0 > a.inverse {
        a != F.0
        F.0 >= a * a.inverse
        false
    }
}

/// The inverse of a positive element is positive.
theorem inverse_of_positive_is_positive[F: OrderedField](a: F) {
    F.0 < a implies F.0 < a.inverse
} by {
    if F.0 < a {
        F.0 <= a
        F.0 <= a.inverse
        a != F.0
        a * a.inverse = F.1
        a.inverse != F.0
        F.0 < a.inverse
    }
}

/// The inverse of a negative element is negative.
theorem inverse_of_negative_is_negative[F: OrderedField](a: F) {
    a < F.0 implies a.inverse < F.0
} by {
    if a < F.0 {
        a != F.0
        a * a.inverse = F.1
        -F.0 < -a
        -F.0 = F.0
        F.0 < -a
        inverse_of_positive_is_positive(-a)
        F.0 < (-a).inverse
        (-a) * -(a.inverse) = a * a.inverse
        (-a) * -(a.inverse) = F.1
        -(a.inverse) = (-a).inverse
        F.0 < -(a.inverse)
        -(-(a.inverse)) < -F.0
        -(-(a.inverse)) = a.inverse
        a.inverse < F.0
    }
}

theorem zpows_of_nonnegative_are_nonnegative[F: OrderedField](a: F, n: Int) {
    F.0 <= a implies F.0 <= a.zpow(n)
} by {
    if n >= Int.0 {}
    else {
        n <= Int.0
        Int.0 <= -n
        -n >= Int.0
        F.0 <= a.zpow(-n)
        F.0 <= a.zpow(-n).inverse
    }
}

theorem odd_pows_of_negative_are_negative[F: OrderedField](a: F, n: Nat) {
    a < F.0 implies a.pow(Nat.2 * n + Nat.1) < F.0
} by {
    a <= F.0
    a.pow(Nat.2 * n) >= F.0
    F.0 <= a.pow(Nat.2 * n)
    a.pow(Nat.2 * n) * a <= F.0 * a
    F.0 * a = F.0
    a.pow(Nat.2 * n) * a <= F.0
    a.pow(Nat.1) = a
    a.pow(Nat.2 * n) * a.pow(Nat.1) = a.pow(Nat.2 * n + Nat.1)
    F.0 != a
    a.pow((Nat.2 * n).suc) != F.0
    a.pow(Nat.2 * n) * a = a * a.pow(Nat.2 * n)
    a * a.pow(Nat.2 * n) = a.pow((Nat.2 * n).suc)
    a.pow(Nat.2 * n) * a != F.0
    a.pow(Nat.2 * n) * a < F.0
}

/// Positive base raised to any natural number power is positive.
theorem positive_pows_of_positive_are_positive[F: OrderedField](a: F, n: Nat) {
    a > F.0 implies a.pow(n) > F.0
} by {
    define p(k: Nat) -> Bool { a.pow(k) > F.0 }
    p(Nat.0)
    forall(k: Nat) {
        if p(k) {
            F.0 <= a
            F.0 <= a.pow(k)
            F.0 <= a * a.pow(k)
            a != F.0
            a.pow(k) != F.0
            a * a.pow(k) != F.0
            a * a.pow(k) > F.0
            a * a.pow(k) = a.pow(k.suc)
            p(k.suc)
        }
    }
}

/// Monotonicity of natural number powers for nonnegative base.
theorem pow_le_pow_of_le[F: OrderedField](a: F, b: F, n: Nat) {
    F.0 <= a and a <= b implies a.pow(n) <= b.pow(n)
} by {
    define p(k: Nat) -> Bool { a.pow(k) <= b.pow(k) }
    p(Nat.0)
    forall(k: Nat) {
        if p(k) {
            a * a.pow(k) <= b * a.pow(k)
            F.0 <= b
            b * a.pow(k) <= b * b.pow(k)
            a * a.pow(k) <= b * b.pow(k)
            a * a.pow(k) = a.pow(k.suc)
            b * b.pow(k) = b.pow(k.suc)
            p(k.suc)
        }
    }
}

theorem nonnegative_odd_zpows_of_negative_are_negative[F: OrderedField](a: F, n: Int) {
    a < F.0 and Int.0 <= n implies a.zpow(Int.2 * n + Int.1) < F.0
} by {
    let k: Nat satisfy {
        n = Int.from_nat(k)
    }
    Int.2 * n + Int.1 = Int.from_nat(Nat.2 * k + Nat.1)
    a.pow(Nat.2 * k + Nat.1) < F.0
    a.zpow(Int.from_nat(Nat.2 * k + Nat.1)) = a.pow(Nat.2 * k + Nat.1)
}

/// Right multiplication by a positive scalar arises from a bundled order isomorphism whose
/// underlying map is right multiplication and whose inverse is right multiplication by the
/// scalar's inverse.
theorem mul_right_order_iso_exists[F: OrderedField](c: F) {
    F.0 < c implies exists(e: OrderIso[F, F]) {
        e.map = mul_right_scalar_map(c) and e.inv = mul_right_scalar_map(c.inverse)
    }
} by {
    if F.0 < c {
        inverse_of_positive_is_positive(c)
        F.0 < c.inverse
        mul_right_scalar_map_is_order_iso_pair_of_positive(c)
        is_order_iso_pair(mul_right_scalar_map(c), mul_right_scalar_map(c.inverse))
        let e: OrderIso[F, F] satisfy {
            OrderIso[F, F].new(mul_right_scalar_map(c), mul_right_scalar_map(c.inverse)) = Option.some(e)
        }
        order_iso_new_map(mul_right_scalar_map(c), mul_right_scalar_map(c.inverse), e)
        order_iso_new_inv(mul_right_scalar_map(c), mul_right_scalar_map(c.inverse), e)
        e.map = mul_right_scalar_map(c) and e.inv = mul_right_scalar_map(c.inverse)
    }
}

/// Left multiplication by a positive scalar arises from a bundled order isomorphism whose
/// underlying map is left multiplication and whose inverse is left multiplication by the
/// scalar's inverse.
theorem mul_left_order_iso_exists[F: OrderedField](c: F) {
    F.0 < c implies exists(e: OrderIso[F, F]) {
        e.map = mul_left_scalar_map(c) and e.inv = mul_left_scalar_map(c.inverse)
    }
} by {
    if F.0 < c {
        inverse_of_positive_is_positive(c)
        F.0 < c.inverse
        mul_left_scalar_map_is_order_iso_pair_of_positive(c)
        is_order_iso_pair(mul_left_scalar_map(c), mul_left_scalar_map(c.inverse))
        let e: OrderIso[F, F] satisfy {
            OrderIso[F, F].new(mul_left_scalar_map(c), mul_left_scalar_map(c.inverse)) = Option.some(e)
        }
        order_iso_new_map(mul_left_scalar_map(c), mul_left_scalar_map(c.inverse), e)
        order_iso_new_inv(mul_left_scalar_map(c), mul_left_scalar_map(c.inverse), e)
        e.map = mul_left_scalar_map(c) and e.inv = mul_left_scalar_map(c.inverse)
    }
}

/// Right multiplication by a negative scalar arises from a bundled order-dual isomorphism whose
/// underlying map is right multiplication and whose inverse is right multiplication by the
/// scalar's inverse.
theorem mul_right_order_dual_iso_exists[F: OrderedField](c: F) {
    c < F.0 implies exists(e: OrderDualIso[F, F]) {
        e.map = mul_right_scalar_map(c) and e.inv = mul_right_scalar_map(c.inverse)
    }
} by {
    if c < F.0 {
        inverse_of_negative_is_negative(c)
        c.inverse < F.0
        mul_right_scalar_map_is_order_dual_iso_pair_of_negative(c)
        is_order_dual_iso_pair(mul_right_scalar_map(c), mul_right_scalar_map(c.inverse))
        let e: OrderDualIso[F, F] satisfy {
            OrderDualIso[F, F].new(mul_right_scalar_map(c), mul_right_scalar_map(c.inverse)) = Option.some(e)
        }
        order_dual_iso_new_map(mul_right_scalar_map(c), mul_right_scalar_map(c.inverse), e)
        order_dual_iso_new_inv(mul_right_scalar_map(c), mul_right_scalar_map(c.inverse), e)
        e.map = mul_right_scalar_map(c) and e.inv = mul_right_scalar_map(c.inverse)
    }
}

/// Left multiplication by a negative scalar arises from a bundled order-dual isomorphism whose
/// underlying map is left multiplication and whose inverse is left multiplication by the
/// scalar's inverse.
theorem mul_left_order_dual_iso_exists[F: OrderedField](c: F) {
    c < F.0 implies exists(e: OrderDualIso[F, F]) {
        e.map = mul_left_scalar_map(c) and e.inv = mul_left_scalar_map(c.inverse)
    }
} by {
    if c < F.0 {
        inverse_of_negative_is_negative(c)
        c.inverse < F.0
        mul_left_scalar_map_is_order_dual_iso_pair_of_negative(c)
        is_order_dual_iso_pair(mul_left_scalar_map(c), mul_left_scalar_map(c.inverse))
        let e: OrderDualIso[F, F] satisfy {
            OrderDualIso[F, F].new(mul_left_scalar_map(c), mul_left_scalar_map(c.inverse)) = Option.some(e)
        }
        order_dual_iso_new_map(mul_left_scalar_map(c), mul_left_scalar_map(c.inverse), e)
        order_dual_iso_new_inv(mul_left_scalar_map(c), mul_left_scalar_map(c.inverse), e)
        e.map = mul_left_scalar_map(c) and e.inv = mul_left_scalar_map(c.inverse)
    }
}
