from nat import Nat, lte_and_lt
from int import Int
from list import List
from semiring import Semiring
from algebra.ring.ring import Ring
from polynomial import Polynomial, polynomial_eval, polynomial_eval_sub,
    polynomial_ext_pointwise, polynomial_sub_coeff, polynomial_support_bounded_by,
    polynomial_support_bounded_by_apply, polynomial_zero_coeff
from polynomial_sub_support import polynomial_sub_support_bounded_by
from polynomial_int_roots import int_roots_on_list,
    int_polynomial_root_list_length_lt_support_bound

numerals Nat

/// True if two polynomials have the same coefficients below `n`.
///
/// The normal form for polynomials known to be supported below `n`: the finite initial segment
/// of coefficients determines the polynomial, so comparing that segment decides identity.
define agrees_below[R: Semiring](p: Polynomial[R], q: Polynomial[R], n: Nat) -> Bool {
    forall(k: Nat) {
        k < n implies p.coeff(k) = q.coeff(k)
    }
}

/// Agreement below the bound follows from equality.
theorem agrees_below_of_eq[R: Semiring](p: Polynomial[R], q: Polynomial[R], n: Nat) {
    p = q implies agrees_below(p, q, n)
} by {
    if p = q {
        forall(k: Nat) {
            (k < n implies p.coeff(k) = q.coeff(k))
        }
        (agrees_below(p, q, n) = forall(j: Nat) {
            j < n implies p.coeff(j) = q.coeff(j)
        })
        agrees_below(p, q, n)
    }
}

/// The agreement condition applies at an index below the bound.
theorem agrees_below_apply[R: Semiring](
    p: Polynomial[R], q: Polynomial[R], n: Nat, k: Nat
) {
    agrees_below(p, q, n) and k < n implies p.coeff(k) = q.coeff(k)
} by {
    if agrees_below(p, q, n) and k < n {
        (agrees_below(p, q, n) = forall(j: Nat) {
            j < n implies p.coeff(j) = q.coeff(j)
        })
        forall(j: Nat) {
            j < n implies p.coeff(j) = q.coeff(j)
        }
        (k < n implies p.coeff(k) = q.coeff(k))
        p.coeff(k) = q.coeff(k)
    }
}

/// Two polynomials supported below `n` agreeing below `n` are equal.
///
/// This is the decision procedure for identity in coefficient form: above the bound both
/// vanish, so the finite initial segment carries all the information.
theorem polynomial_eq_of_agrees_below[R: Semiring](
    p: Polynomial[R], q: Polynomial[R], n: Nat
) {
    polynomial_support_bounded_by(p, n) and polynomial_support_bounded_by(q, n)
        and agrees_below(p, q, n)
        implies p = q
} by {
    if polynomial_support_bounded_by(p, n) and polynomial_support_bounded_by(q, n)
        and agrees_below(p, q, n) {
        forall(k: Nat) {
            if k < n {
                agrees_below_apply(p, q, n, k)
                p.coeff(k) = q.coeff(k)
            }
            if not k < n {
                polynomial_support_bounded_by_apply(p, n, k)
                p.coeff(k) = R.0
                polynomial_support_bounded_by_apply(q, n, k)
                q.coeff(k) = R.0
                p.coeff(k) = q.coeff(k)
            }
            p.coeff(k) = q.coeff(k)
        }
        polynomial_ext_pointwise(p, q)
        p = q
    }
}

/// Identity of polynomials supported below `n` is exactly agreement below `n`.
theorem polynomial_eq_iff_agrees_below[R: Semiring](
    p: Polynomial[R], q: Polynomial[R], n: Nat
) {
    polynomial_support_bounded_by(p, n) and polynomial_support_bounded_by(q, n)
        implies ((p = q) = agrees_below(p, q, n))
} by {
    if polynomial_support_bounded_by(p, n) and polynomial_support_bounded_by(q, n) {
        if p = q {
            agrees_below_of_eq(p, q, n)
            agrees_below(p, q, n)
        }
        if agrees_below(p, q, n) {
            polynomial_eq_of_agrees_below(p, q, n)
            p = q
        }
        ((p = q) implies agrees_below(p, q, n))
        (agrees_below(p, q, n) implies (p = q))
        ((p = q) = agrees_below(p, q, n))
    }
}

/// True if two integer polynomials take the same value at every entry of a list.
define int_agrees_on_list(
    p: Polynomial[Int], q: Polynomial[Int], points: List[Int]
) -> Bool {
    forall(x: Int) {
        points.contains(x) implies polynomial_eval(p, x) = polynomial_eval(q, x)
    }
}

/// Points of agreement are roots of the difference.
theorem int_agreement_gives_roots(
    p: Polynomial[Int], q: Polynomial[Int], points: List[Int]
) {
    int_agrees_on_list(p, q, points) implies int_roots_on_list(p.sub(q), points)
} by {
    if int_agrees_on_list(p, q, points) {
        (int_agrees_on_list(p, q, points) = forall(y: Int) {
            points.contains(y) implies polynomial_eval(p, y) = polynomial_eval(q, y)
        })
        forall(x: Int) {
            if points.contains(x) {
                (points.contains(x)
                    implies polynomial_eval(p, x) = polynomial_eval(q, x))
                polynomial_eval(p, x) = polynomial_eval(q, x)
                polynomial_eval_sub(p, q, x)
                (polynomial_eval(p.sub(q), x)
                    = polynomial_eval(p, x) - polynomial_eval(q, x))
                polynomial_eval(p, x) - polynomial_eval(q, x) = Int.0
                polynomial_eval(p.sub(q), x) = Int.0
            }
            (points.contains(x) implies polynomial_eval(p.sub(q), x) = Int.0)
        }
        (int_roots_on_list(p.sub(q), points) = forall(y: Int) {
            points.contains(y) implies polynomial_eval(p.sub(q), y) = Int.0
        })
        int_roots_on_list(p.sub(q), points)
    }
}

/// A difference of integer polynomials with no nonzero coefficient is zero.
theorem int_polynomial_sub_zero_of_eq(p: Polynomial[Int], q: Polynomial[Int]) {
    p = q implies p.sub(q) = Polynomial[Int].zero
} by {
    if p = q {
        forall(k: Nat) {
            polynomial_sub_coeff(p, q, k)
            p.sub(q).coeff(k) = p.coeff(k) + -q.coeff(k)
            p.coeff(k) + -q.coeff(k) = Int.0
            p.sub(q).coeff(k) = Int.0
            polynomial_zero_coeff[Int](k)
            Polynomial[Int].zero.coeff(k) = Int.0
            p.sub(q).coeff(k) = Polynomial[Int].zero.coeff(k)
        }
        polynomial_ext_pointwise(p.sub(q), Polynomial[Int].zero)
        p.sub(q) = Polynomial[Int].zero
    }
}

/// A zero difference makes two polynomials equal.
theorem int_polynomial_eq_of_sub_zero(p: Polynomial[Int], q: Polynomial[Int]) {
    p.sub(q) = Polynomial[Int].zero implies p = q
} by {
    if p.sub(q) = Polynomial[Int].zero {
        forall(k: Nat) {
            polynomial_sub_coeff(p, q, k)
            p.sub(q).coeff(k) = p.coeff(k) + -q.coeff(k)
            polynomial_zero_coeff[Int](k)
            Polynomial[Int].zero.coeff(k) = Int.0
            p.sub(q).coeff(k) = Int.0
            p.coeff(k) + -q.coeff(k) = Int.0
            p.coeff(k) = q.coeff(k)
        }
        polynomial_ext_pointwise(p, q)
        p = q
    }
}

/// Two integer polynomials supported below `n` agreeing at `n` distinct points are equal.
///
/// The evaluation form of the identity test. The difference is supported below `n` as well, so
/// if it were nonzero it would have fewer than `n` distinct roots; `n` points of agreement is
/// therefore already too many. This is why `n` evaluations decide an identity between
/// polynomials of degree below `n`, with no bound on how large the coefficients are.
theorem int_polynomial_eq_of_agreement(
    p: Polynomial[Int], q: Polynomial[Int], points: List[Int], n: Nat
) {
    polynomial_support_bounded_by(p, n) and polynomial_support_bounded_by(q, n)
        and points.is_unique and n <= points.length and int_agrees_on_list(p, q, points)
        implies p = q
} by {
    if polynomial_support_bounded_by(p, n) and polynomial_support_bounded_by(q, n)
        and points.is_unique and n <= points.length and int_agrees_on_list(p, q, points) {
        polynomial_sub_support_bounded_by(p, q, n)
        polynomial_support_bounded_by(p.sub(q), n)
        int_agreement_gives_roots(p, q, points)
        int_roots_on_list(p.sub(q), points)
        if p.sub(q) != Polynomial[Int].zero {
            int_polynomial_root_list_length_lt_support_bound(p.sub(q), points, n)
            points.length < n
            lte_and_lt(n, points.length, n)
            n < n
            false
        }
        p.sub(q) = Polynomial[Int].zero
        int_polynomial_eq_of_sub_zero(p, q)
        p = q
    }
}
