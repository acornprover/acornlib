/// Bounds and extrema for functions with partially ordered values.

from order import PartialOrder, lte_antisymm
from data.basic.set import Set, sets_subset_intersection, sets_subset_union, difference_subset

/// The value `bound` is an upper bound for `f` on `domain`.
define is_upper_bound_on[Domain, Value: PartialOrder](
    domain: Set[Domain], f: Domain -> Value, bound: Value
) -> Bool {
    forall(point: Domain) {
        domain.contains(point) implies f(point) <= bound
    }
}

/// The value `bound` is a lower bound for `f` on `domain`.
define is_lower_bound_on[Domain, Value: PartialOrder](
    domain: Set[Domain], f: Domain -> Value, bound: Value
) -> Bool {
    forall(point: Domain) {
        domain.contains(point) implies bound <= f(point)
    }
}

/// The function `f` has an upper bound on `domain`.
define is_bounded_above_on[Domain, Value: PartialOrder](
    domain: Set[Domain], f: Domain -> Value
) -> Bool {
    exists(bound: Value) {
        is_upper_bound_on(domain, f, bound)
    }
}

/// The function `f` has a lower bound on `domain`.
define is_bounded_below_on[Domain, Value: PartialOrder](
    domain: Set[Domain], f: Domain -> Value
) -> Bool {
    exists(bound: Value) {
        is_lower_bound_on(domain, f, bound)
    }
}

/// The function `f` has both an upper and a lower bound on `domain`.
define is_bounded_on[Domain, Value: PartialOrder](
    domain: Set[Domain], f: Domain -> Value
) -> Bool {
    is_bounded_above_on(domain, f) and is_bounded_below_on(domain, f)
}

/// The value `bound` is an upper bound for `f` on both domains.
define is_common_upper_bound_on[Domain, Value: PartialOrder](
    left: Set[Domain], right: Set[Domain], f: Domain -> Value, bound: Value
) -> Bool {
    is_upper_bound_on(left, f, bound) and is_upper_bound_on(right, f, bound)
}

/// The value `bound` is a lower bound for `f` on both domains.
define is_common_lower_bound_on[Domain, Value: PartialOrder](
    left: Set[Domain], right: Set[Domain], f: Domain -> Value, bound: Value
) -> Bool {
    is_lower_bound_on(left, f, bound) and is_lower_bound_on(right, f, bound)
}

/// The function `f` has a common upper bound on both domains.
define is_common_bounded_above_on[Domain, Value: PartialOrder](
    left: Set[Domain], right: Set[Domain], f: Domain -> Value
) -> Bool {
    exists(bound: Value) {
        is_common_upper_bound_on(left, right, f, bound)
    }
}

/// The function `f` has a common lower bound on both domains.
define is_common_bounded_below_on[Domain, Value: PartialOrder](
    left: Set[Domain], right: Set[Domain], f: Domain -> Value
) -> Bool {
    exists(bound: Value) {
        is_common_lower_bound_on(left, right, f, bound)
    }
}

/// The function `f` has both common upper and common lower bounds on both domains.
define is_common_bounded_on[Domain, Value: PartialOrder](
    left: Set[Domain], right: Set[Domain], f: Domain -> Value
) -> Bool {
    is_common_bounded_above_on(left, right, f) and is_common_bounded_below_on(left, right, f)
}

/// An upper bound dominates the value at every point of its domain.
theorem upper_bound_on_apply[Domain, Value: PartialOrder](
    domain: Set[Domain], f: Domain -> Value, bound: Value, point: Domain
) {
    is_upper_bound_on(domain, f, bound) and domain.contains(point) implies f(point) <= bound
} by {
    if is_upper_bound_on(domain, f, bound) and domain.contains(point) {
        is_upper_bound_on(domain, f, bound) = forall(candidate: Domain) {
            domain.contains(candidate) implies f(candidate) <= bound
        }
        domain.contains(point) implies f(point) <= bound
        f(point) <= bound
    }
}

/// A lower bound is dominated by the value at every point of its domain.
theorem lower_bound_on_apply[Domain, Value: PartialOrder](
    domain: Set[Domain], f: Domain -> Value, bound: Value, point: Domain
) {
    is_lower_bound_on(domain, f, bound) and domain.contains(point) implies bound <= f(point)
} by {
    if is_lower_bound_on(domain, f, bound) and domain.contains(point) {
        is_lower_bound_on(domain, f, bound) = forall(candidate: Domain) {
            domain.contains(candidate) implies bound <= f(candidate)
        }
        domain.contains(point) implies bound <= f(point)
        bound <= f(point)
    }
}

/// If `f` is pointwise below `g` on the domain, any upper bound for `g` bounds `f`.
theorem upper_bound_on_of_pointwise_le[Domain, Value: PartialOrder](
    domain: Set[Domain], f: Domain -> Value, g: Domain -> Value, bound: Value
) {
    (forall(point: Domain) { domain.contains(point) implies f(point) <= g(point) })
    and is_upper_bound_on(domain, g, bound)
    implies is_upper_bound_on(domain, f, bound)
} by {
    if (forall(point: Domain) { domain.contains(point) implies f(point) <= g(point) })
    and is_upper_bound_on(domain, g, bound) {
        forall(point: Domain) {
            if domain.contains(point) {
                f(point) <= g(point)
                upper_bound_on_apply(domain, g, bound, point)
                g(point) <= bound
                f(point) <= bound
            }
        }
    }
}

/// If `f` is pointwise above `g` on the domain, any lower bound for `g` bounds `f`.
theorem lower_bound_on_of_pointwise_ge[Domain, Value: PartialOrder](
    domain: Set[Domain], f: Domain -> Value, g: Domain -> Value, bound: Value
) {
    (forall(point: Domain) { domain.contains(point) implies g(point) <= f(point) })
    and is_lower_bound_on(domain, g, bound)
    implies is_lower_bound_on(domain, f, bound)
} by {
    if (forall(point: Domain) { domain.contains(point) implies g(point) <= f(point) })
    and is_lower_bound_on(domain, g, bound) {
        forall(point: Domain) {
            if domain.contains(point) {
                lower_bound_on_apply(domain, g, bound, point)
                bound <= g(point)
                g(point) <= f(point)
                bound <= f(point)
            }
        }
    }
}

/// Pointwise domination transfers bounded-above witnesses from the larger function to the smaller one.
theorem bounded_above_on_of_pointwise_le[Domain, Value: PartialOrder](
    domain: Set[Domain], f: Domain -> Value, g: Domain -> Value
) {
    (forall(point: Domain) { domain.contains(point) implies f(point) <= g(point) })
    and is_bounded_above_on(domain, g)
    implies is_bounded_above_on(domain, f)
} by {
    if (forall(point: Domain) { domain.contains(point) implies f(point) <= g(point) })
    and is_bounded_above_on(domain, g) {
        let bound: Value satisfy {
            is_upper_bound_on(domain, g, bound)
        }
        upper_bound_on_of_pointwise_le(domain, f, g, bound)
        is_upper_bound_on(domain, f, bound)
        exists(witness: Value) {
            is_upper_bound_on(domain, f, witness)
        }
    }
}

/// Pointwise reverse domination transfers bounded-below witnesses from the smaller function to the larger one.
theorem bounded_below_on_of_pointwise_ge[Domain, Value: PartialOrder](
    domain: Set[Domain], f: Domain -> Value, g: Domain -> Value
) {
    (forall(point: Domain) { domain.contains(point) implies g(point) <= f(point) })
    and is_bounded_below_on(domain, g)
    implies is_bounded_below_on(domain, f)
} by {
    if (forall(point: Domain) { domain.contains(point) implies g(point) <= f(point) })
    and is_bounded_below_on(domain, g) {
        let bound: Value satisfy {
            is_lower_bound_on(domain, g, bound)
        }
        lower_bound_on_of_pointwise_ge(domain, f, g, bound)
        is_lower_bound_on(domain, f, bound)
        exists(witness: Value) {
            is_lower_bound_on(domain, f, witness)
        }
    }
}

/// The function `f` attains a maximum at `point` on `domain`.
define attains_maximum_at[Domain, Value: PartialOrder](
    domain: Set[Domain], f: Domain -> Value, point: Domain
) -> Bool {
    domain.contains(point) and is_upper_bound_on(domain, f, f(point))
}

/// The function `f` attains a minimum at `point` on `domain`.
define attains_minimum_at[Domain, Value: PartialOrder](
    domain: Set[Domain], f: Domain -> Value, point: Domain
) -> Bool {
    domain.contains(point) and is_lower_bound_on(domain, f, f(point))
}

/// The function `f` attains a maximum somewhere on `domain`.
define attains_maximum_on[Domain, Value: PartialOrder](
    domain: Set[Domain], f: Domain -> Value
) -> Bool {
    exists(point: Domain) {
        attains_maximum_at(domain, f, point)
    }
}

/// The function `f` attains a minimum somewhere on `domain`.
define attains_minimum_on[Domain, Value: PartialOrder](
    domain: Set[Domain], f: Domain -> Value
) -> Bool {
    exists(point: Domain) {
        attains_minimum_at(domain, f, point)
    }
}

/// A point where a function attains a maximum belongs to the domain.
theorem maximum_point_in_domain[Domain, Value: PartialOrder](
    domain: Set[Domain], f: Domain -> Value, point: Domain
) {
    attains_maximum_at(domain, f, point) implies domain.contains(point)
} by {
    attains_maximum_at(domain, f, point)
    domain.contains(point)
}

/// A point where a function attains a minimum belongs to the domain.
theorem minimum_point_in_domain[Domain, Value: PartialOrder](
    domain: Set[Domain], f: Domain -> Value, point: Domain
) {
    attains_minimum_at(domain, f, point) implies domain.contains(point)
} by {
    attains_minimum_at(domain, f, point)
    domain.contains(point)
}

/// A maximum value is an upper bound.
theorem maximum_value_is_upper_bound_on[Domain, Value: PartialOrder](
    domain: Set[Domain], f: Domain -> Value, point: Domain
) {
    attains_maximum_at(domain, f, point) implies is_upper_bound_on(domain, f, f(point))
} by {
    if attains_maximum_at(domain, f, point) {
        is_upper_bound_on(domain, f, f(point))
    }
}

/// A minimum value is a lower bound.
theorem minimum_value_is_lower_bound_on[Domain, Value: PartialOrder](
    domain: Set[Domain], f: Domain -> Value, point: Domain
) {
    attains_minimum_at(domain, f, point) implies is_lower_bound_on(domain, f, f(point))
} by {
    if attains_minimum_at(domain, f, point) {
        is_lower_bound_on(domain, f, f(point))
    }
}

/// Any two maximum points have the same function value.
theorem attains_maximum_value_unique[Domain, Value: PartialOrder](
    domain: Set[Domain], f: Domain -> Value, left_point: Domain, right_point: Domain
) {
    attains_maximum_at(domain, f, left_point) and attains_maximum_at(domain, f, right_point)
    implies f(left_point) = f(right_point)
} by {
    if attains_maximum_at(domain, f, left_point) and attains_maximum_at(domain, f, right_point) {
        maximum_value_is_upper_bound_on(domain, f, left_point)
        is_upper_bound_on(domain, f, f(left_point))
        maximum_value_is_upper_bound_on(domain, f, right_point)
        is_upper_bound_on(domain, f, f(right_point))
        maximum_point_in_domain(domain, f, left_point)
        domain.contains(left_point)
        maximum_point_in_domain(domain, f, right_point)
        domain.contains(right_point)
        upper_bound_on_apply(domain, f, f(left_point), right_point)
        f(right_point) <= f(left_point)
        upper_bound_on_apply(domain, f, f(right_point), left_point)
        f(left_point) <= f(right_point)
        lte_antisymm(f(left_point), f(right_point))
        f(left_point) = f(right_point)
    }
}

/// Any two minimum points have the same function value.
theorem attains_minimum_value_unique[Domain, Value: PartialOrder](
    domain: Set[Domain], f: Domain -> Value, left_point: Domain, right_point: Domain
) {
    attains_minimum_at(domain, f, left_point) and attains_minimum_at(domain, f, right_point)
    implies f(left_point) = f(right_point)
} by {
    if attains_minimum_at(domain, f, left_point) and attains_minimum_at(domain, f, right_point) {
        minimum_value_is_lower_bound_on(domain, f, left_point)
        is_lower_bound_on(domain, f, f(left_point))
        minimum_value_is_lower_bound_on(domain, f, right_point)
        is_lower_bound_on(domain, f, f(right_point))
        minimum_point_in_domain(domain, f, left_point)
        domain.contains(left_point)
        minimum_point_in_domain(domain, f, right_point)
        domain.contains(right_point)
        lower_bound_on_apply(domain, f, f(left_point), right_point)
        f(left_point) <= f(right_point)
        lower_bound_on_apply(domain, f, f(right_point), left_point)
        f(right_point) <= f(left_point)
        lte_antisymm(f(left_point), f(right_point))
        f(left_point) = f(right_point)
    }
}

/// A function which attains a maximum is bounded above.
theorem attains_maximum_imp_bounded_above[Domain, Value: PartialOrder](
    domain: Set[Domain], f: Domain -> Value
) {
    attains_maximum_on(domain, f) implies is_bounded_above_on(domain, f)
} by {
    if attains_maximum_on(domain, f) {
        let point: Domain satisfy {
            attains_maximum_at(domain, f, point)
        }
        maximum_value_is_upper_bound_on(domain, f, point)
        is_upper_bound_on(domain, f, f(point))
        exists(bound: Value) {
            is_upper_bound_on(domain, f, bound)
        }
        is_bounded_above_on(domain, f)
    }
}

/// A function which attains a minimum is bounded below.
theorem attains_minimum_imp_bounded_below[Domain, Value: PartialOrder](
    domain: Set[Domain], f: Domain -> Value
) {
    attains_minimum_on(domain, f) implies is_bounded_below_on(domain, f)
} by {
    if attains_minimum_on(domain, f) {
        let point: Domain satisfy {
            attains_minimum_at(domain, f, point)
        }
        minimum_value_is_lower_bound_on(domain, f, point)
        is_lower_bound_on(domain, f, f(point))
        exists(bound: Value) {
            is_lower_bound_on(domain, f, bound)
        }
        is_bounded_below_on(domain, f)
    }
}

/// A function which attains both extrema is bounded.
theorem attains_extrema_imp_bounded[Domain, Value: PartialOrder](
    domain: Set[Domain], f: Domain -> Value
) {
    attains_maximum_on(domain, f) and attains_minimum_on(domain, f)
    implies is_bounded_on(domain, f)
} by {
    if attains_maximum_on(domain, f) and attains_minimum_on(domain, f) {
        attains_maximum_imp_bounded_above(domain, f)
        attains_minimum_imp_bounded_below(domain, f)
        is_bounded_above_on(domain, f)
        is_bounded_below_on(domain, f)
        is_bounded_above_on(domain, f) and is_bounded_below_on(domain, f)
        is_bounded_on(domain, f)
    }
}

/// An upper bound on a set remains an upper bound after restricting the domain.
theorem upper_bound_on_subset[Domain, Value: PartialOrder](
    subset: Set[Domain], superset: Set[Domain], f: Domain -> Value, bound: Value
) {
    subset.subset(superset) and is_upper_bound_on(superset, f, bound)
    implies is_upper_bound_on(subset, f, bound)
} by {
    if subset.subset(superset) and is_upper_bound_on(superset, f, bound) {
        forall(point: Domain) {
            if subset.contains(point) {
                subset.subset(superset) = forall(candidate: Domain) {
                    subset.contains(candidate) implies superset.contains(candidate)
                }
                subset.contains(point) implies superset.contains(point)
                superset.contains(point)
                upper_bound_on_apply(superset, f, bound, point)
                f(point) <= bound
            }
        }
        is_upper_bound_on(subset, f, bound)
    }
}

/// A lower bound on a set remains a lower bound after restricting the domain.
theorem lower_bound_on_subset[Domain, Value: PartialOrder](
    subset: Set[Domain], superset: Set[Domain], f: Domain -> Value, bound: Value
) {
    subset.subset(superset) and is_lower_bound_on(superset, f, bound)
    implies is_lower_bound_on(subset, f, bound)
} by {
    if subset.subset(superset) and is_lower_bound_on(superset, f, bound) {
        forall(point: Domain) {
            if subset.contains(point) {
                subset.subset(superset) = forall(candidate: Domain) {
                    subset.contains(candidate) implies superset.contains(candidate)
                }
                subset.contains(point) implies superset.contains(point)
                superset.contains(point)
                lower_bound_on_apply(superset, f, bound, point)
                bound <= f(point)
            }
        }
        is_lower_bound_on(subset, f, bound)
    }
}

/// A maximum attained on a set remains a maximum on any subset containing its point.
theorem attains_maximum_at_subset[Domain, Value: PartialOrder](
    subset: Set[Domain], superset: Set[Domain], f: Domain -> Value, point: Domain
) {
    subset.subset(superset) and subset.contains(point) and attains_maximum_at(superset, f, point)
    implies attains_maximum_at(subset, f, point)
} by {
    maximum_value_is_upper_bound_on(superset, f, point)
    is_upper_bound_on(superset, f, f(point))
    upper_bound_on_subset(subset, superset, f, f(point))
    is_upper_bound_on(subset, f, f(point))
    subset.contains(point) and is_upper_bound_on(subset, f, f(point))
    attains_maximum_at(subset, f, point)
}

/// A minimum attained on a set remains a minimum on any subset containing its point.
theorem attains_minimum_at_subset[Domain, Value: PartialOrder](
    subset: Set[Domain], superset: Set[Domain], f: Domain -> Value, point: Domain
) {
    subset.subset(superset) and subset.contains(point) and attains_minimum_at(superset, f, point)
    implies attains_minimum_at(subset, f, point)
} by {
    minimum_value_is_lower_bound_on(superset, f, point)
    is_lower_bound_on(superset, f, f(point))
    lower_bound_on_subset(subset, superset, f, f(point))
    is_lower_bound_on(subset, f, f(point))
    subset.contains(point) and is_lower_bound_on(subset, f, f(point))
    attains_minimum_at(subset, f, point)
}

/// A subset containing a maximum point has an attained maximum.
theorem attains_maximum_on_subset_from_point[Domain, Value: PartialOrder](
    subset: Set[Domain], superset: Set[Domain], f: Domain -> Value, point: Domain
) {
    subset.subset(superset) and subset.contains(point) and attains_maximum_at(superset, f, point)
    implies attains_maximum_on(subset, f)
} by {
    attains_maximum_at_subset(subset, superset, f, point)
    attains_maximum_at(subset, f, point)
    exists(witness: Domain) {
        attains_maximum_at(subset, f, witness)
    }
    attains_maximum_on(subset, f)
}

/// A subset containing a minimum point has an attained minimum.
theorem attains_minimum_on_subset_from_point[Domain, Value: PartialOrder](
    subset: Set[Domain], superset: Set[Domain], f: Domain -> Value, point: Domain
) {
    subset.subset(superset) and subset.contains(point) and attains_minimum_at(superset, f, point)
    implies attains_minimum_on(subset, f)
} by {
    attains_minimum_at_subset(subset, superset, f, point)
    attains_minimum_at(subset, f, point)
    exists(witness: Domain) {
        attains_minimum_at(subset, f, witness)
    }
    attains_minimum_on(subset, f)
}

/// Boundedness above is preserved by restricting the domain.
theorem bounded_above_on_subset[Domain, Value: PartialOrder](
    subset: Set[Domain], superset: Set[Domain], f: Domain -> Value
) {
    subset.subset(superset) and is_bounded_above_on(superset, f)
    implies is_bounded_above_on(subset, f)
} by {
    if subset.subset(superset) and is_bounded_above_on(superset, f) {
        let bound: Value satisfy {
            is_upper_bound_on(superset, f, bound)
        }
        upper_bound_on_subset(subset, superset, f, bound)
        is_upper_bound_on(subset, f, bound)
        exists(subset_bound: Value) {
            is_upper_bound_on(subset, f, subset_bound)
        }
        is_bounded_above_on(subset, f)
    }
}

/// Boundedness below is preserved by restricting the domain.
theorem bounded_below_on_subset[Domain, Value: PartialOrder](
    subset: Set[Domain], superset: Set[Domain], f: Domain -> Value
) {
    subset.subset(superset) and is_bounded_below_on(superset, f)
    implies is_bounded_below_on(subset, f)
} by {
    if subset.subset(superset) and is_bounded_below_on(superset, f) {
        let bound: Value satisfy {
            is_lower_bound_on(superset, f, bound)
        }
        lower_bound_on_subset(subset, superset, f, bound)
        is_lower_bound_on(subset, f, bound)
        exists(subset_bound: Value) {
            is_lower_bound_on(subset, f, subset_bound)
        }
        is_bounded_below_on(subset, f)
    }
}

/// Boundedness is preserved by restricting the domain.
theorem bounded_on_subset[Domain, Value: PartialOrder](
    subset: Set[Domain], superset: Set[Domain], f: Domain -> Value
) {
    subset.subset(superset) and is_bounded_on(superset, f)
    implies is_bounded_on(subset, f)
} by {
    if subset.subset(superset) and is_bounded_on(superset, f) {
        bounded_above_on_subset(subset, superset, f)
        bounded_below_on_subset(subset, superset, f)
        is_bounded_above_on(subset, f)
        is_bounded_below_on(subset, f)
        is_bounded_above_on(subset, f) and is_bounded_below_on(subset, f)
        is_bounded_on(subset, f)
    }
}

/// An upper bound on the left set bounds the function on the intersection.
theorem upper_bound_on_intersection_left[Domain, Value: PartialOrder](
    left: Set[Domain], right: Set[Domain], f: Domain -> Value, bound: Value
) {
    is_upper_bound_on(left, f, bound) implies is_upper_bound_on(left.intersection(right), f, bound)
} by {
    sets_subset_intersection(left, right)
    left.intersection(right).subset(left)
    upper_bound_on_subset(left.intersection(right), left, f, bound)
    is_upper_bound_on(left.intersection(right), f, bound)
}

/// An upper bound on the right set bounds the function on the intersection.
theorem upper_bound_on_intersection_right[Domain, Value: PartialOrder](
    left: Set[Domain], right: Set[Domain], f: Domain -> Value, bound: Value
) {
    is_upper_bound_on(right, f, bound) implies is_upper_bound_on(left.intersection(right), f, bound)
} by {
    sets_subset_intersection(left, right)
    left.intersection(right).subset(right)
    upper_bound_on_subset(left.intersection(right), right, f, bound)
    is_upper_bound_on(left.intersection(right), f, bound)
}

/// A lower bound on the left set bounds the function on the intersection.
theorem lower_bound_on_intersection_left[Domain, Value: PartialOrder](
    left: Set[Domain], right: Set[Domain], f: Domain -> Value, bound: Value
) {
    is_lower_bound_on(left, f, bound) implies is_lower_bound_on(left.intersection(right), f, bound)
} by {
    sets_subset_intersection(left, right)
    left.intersection(right).subset(left)
    lower_bound_on_subset(left.intersection(right), left, f, bound)
    is_lower_bound_on(left.intersection(right), f, bound)
}

/// A lower bound on the right set bounds the function on the intersection.
theorem lower_bound_on_intersection_right[Domain, Value: PartialOrder](
    left: Set[Domain], right: Set[Domain], f: Domain -> Value, bound: Value
) {
    is_lower_bound_on(right, f, bound) implies is_lower_bound_on(left.intersection(right), f, bound)
} by {
    sets_subset_intersection(left, right)
    left.intersection(right).subset(right)
    lower_bound_on_subset(left.intersection(right), right, f, bound)
    is_lower_bound_on(left.intersection(right), f, bound)
}

/// Boundedness above on the left set restricts to the intersection.
theorem bounded_above_on_intersection_left[Domain, Value: PartialOrder](
    left: Set[Domain], right: Set[Domain], f: Domain -> Value
) {
    is_bounded_above_on(left, f) implies is_bounded_above_on(left.intersection(right), f)
} by {
    sets_subset_intersection(left, right)
    left.intersection(right).subset(left)
    bounded_above_on_subset(left.intersection(right), left, f)
    is_bounded_above_on(left.intersection(right), f)
}

/// Boundedness above on the right set restricts to the intersection.
theorem bounded_above_on_intersection_right[Domain, Value: PartialOrder](
    left: Set[Domain], right: Set[Domain], f: Domain -> Value
) {
    is_bounded_above_on(right, f) implies is_bounded_above_on(left.intersection(right), f)
} by {
    sets_subset_intersection(left, right)
    left.intersection(right).subset(right)
    bounded_above_on_subset(left.intersection(right), right, f)
    is_bounded_above_on(left.intersection(right), f)
}

/// Boundedness below on the left set restricts to the intersection.
theorem bounded_below_on_intersection_left[Domain, Value: PartialOrder](
    left: Set[Domain], right: Set[Domain], f: Domain -> Value
) {
    is_bounded_below_on(left, f) implies is_bounded_below_on(left.intersection(right), f)
} by {
    sets_subset_intersection(left, right)
    left.intersection(right).subset(left)
    bounded_below_on_subset(left.intersection(right), left, f)
    is_bounded_below_on(left.intersection(right), f)
}

/// Boundedness below on the right set restricts to the intersection.
theorem bounded_below_on_intersection_right[Domain, Value: PartialOrder](
    left: Set[Domain], right: Set[Domain], f: Domain -> Value
) {
    is_bounded_below_on(right, f) implies is_bounded_below_on(left.intersection(right), f)
} by {
    sets_subset_intersection(left, right)
    left.intersection(right).subset(right)
    bounded_below_on_subset(left.intersection(right), right, f)
    is_bounded_below_on(left.intersection(right), f)
}

/// Boundedness on the left set restricts to the intersection.
theorem bounded_on_intersection_left[Domain, Value: PartialOrder](
    left: Set[Domain], right: Set[Domain], f: Domain -> Value
) {
    is_bounded_on(left, f) implies is_bounded_on(left.intersection(right), f)
} by {
    sets_subset_intersection(left, right)
    left.intersection(right).subset(left)
    bounded_on_subset(left.intersection(right), left, f)
    is_bounded_on(left.intersection(right), f)
}

/// Boundedness on the right set restricts to the intersection.
theorem bounded_on_intersection_right[Domain, Value: PartialOrder](
    left: Set[Domain], right: Set[Domain], f: Domain -> Value
) {
    is_bounded_on(right, f) implies is_bounded_on(left.intersection(right), f)
} by {
    sets_subset_intersection(left, right)
    left.intersection(right).subset(right)
    bounded_on_subset(left.intersection(right), right, f)
    is_bounded_on(left.intersection(right), f)
}

/// A maximum on the left set remains a maximum on the intersection when its point belongs to the intersection.
theorem attains_maximum_at_intersection_left[Domain, Value: PartialOrder](
    left: Set[Domain], right: Set[Domain], f: Domain -> Value, point: Domain
) {
    left.intersection(right).contains(point) and attains_maximum_at(left, f, point)
    implies attains_maximum_at(left.intersection(right), f, point)
} by {
    sets_subset_intersection(left, right)
    left.intersection(right).subset(left)
    attains_maximum_at_subset(left.intersection(right), left, f, point)
    attains_maximum_at(left.intersection(right), f, point)
}

/// A minimum on the left set remains a minimum on the intersection when its point belongs to the intersection.
theorem attains_minimum_at_intersection_left[Domain, Value: PartialOrder](
    left: Set[Domain], right: Set[Domain], f: Domain -> Value, point: Domain
) {
    left.intersection(right).contains(point) and attains_minimum_at(left, f, point)
    implies attains_minimum_at(left.intersection(right), f, point)
} by {
    sets_subset_intersection(left, right)
    left.intersection(right).subset(left)
    attains_minimum_at_subset(left.intersection(right), left, f, point)
    attains_minimum_at(left.intersection(right), f, point)
}

/// A maximum on the right set remains a maximum on the intersection when its point belongs to the intersection.
theorem attains_maximum_at_intersection_right[Domain, Value: PartialOrder](
    left: Set[Domain], right: Set[Domain], f: Domain -> Value, point: Domain
) {
    left.intersection(right).contains(point) and attains_maximum_at(right, f, point)
    implies attains_maximum_at(left.intersection(right), f, point)
} by {
    sets_subset_intersection(left, right)
    left.intersection(right).subset(right)
    attains_maximum_at_subset(left.intersection(right), right, f, point)
    attains_maximum_at(left.intersection(right), f, point)
}

/// A minimum on the right set remains a minimum on the intersection when its point belongs to the intersection.
theorem attains_minimum_at_intersection_right[Domain, Value: PartialOrder](
    left: Set[Domain], right: Set[Domain], f: Domain -> Value, point: Domain
) {
    left.intersection(right).contains(point) and attains_minimum_at(right, f, point)
    implies attains_minimum_at(left.intersection(right), f, point)
} by {
    sets_subset_intersection(left, right)
    left.intersection(right).subset(right)
    attains_minimum_at_subset(left.intersection(right), right, f, point)
    attains_minimum_at(left.intersection(right), f, point)
}

/// An upper bound on a set bounds the function on any difference from that set.
theorem upper_bound_on_difference[Domain, Value: PartialOrder](
    domain: Set[Domain], excluded: Set[Domain], f: Domain -> Value, bound: Value
) {
    is_upper_bound_on(domain, f, bound) implies is_upper_bound_on(domain.difference(excluded), f, bound)
} by {
    difference_subset(domain, excluded)
    upper_bound_on_subset(domain.difference(excluded), domain, f, bound)
    is_upper_bound_on(domain.difference(excluded), f, bound)
}

/// A lower bound on a set bounds the function on any difference from that set.
theorem lower_bound_on_difference[Domain, Value: PartialOrder](
    domain: Set[Domain], excluded: Set[Domain], f: Domain -> Value, bound: Value
) {
    is_lower_bound_on(domain, f, bound) implies is_lower_bound_on(domain.difference(excluded), f, bound)
} by {
    difference_subset(domain, excluded)
    lower_bound_on_subset(domain.difference(excluded), domain, f, bound)
    is_lower_bound_on(domain.difference(excluded), f, bound)
}

/// Boundedness above restricts to any difference from the domain.
theorem bounded_above_on_difference[Domain, Value: PartialOrder](
    domain: Set[Domain], excluded: Set[Domain], f: Domain -> Value
) {
    is_bounded_above_on(domain, f) implies is_bounded_above_on(domain.difference(excluded), f)
} by {
    difference_subset(domain, excluded)
    bounded_above_on_subset(domain.difference(excluded), domain, f)
    is_bounded_above_on(domain.difference(excluded), f)
}

/// Boundedness below restricts to any difference from the domain.
theorem bounded_below_on_difference[Domain, Value: PartialOrder](
    domain: Set[Domain], excluded: Set[Domain], f: Domain -> Value
) {
    is_bounded_below_on(domain, f) implies is_bounded_below_on(domain.difference(excluded), f)
} by {
    difference_subset(domain, excluded)
    bounded_below_on_subset(domain.difference(excluded), domain, f)
    is_bounded_below_on(domain.difference(excluded), f)
}

/// Boundedness restricts to any difference from the domain.
theorem bounded_on_difference[Domain, Value: PartialOrder](
    domain: Set[Domain], excluded: Set[Domain], f: Domain -> Value
) {
    is_bounded_on(domain, f) implies is_bounded_on(domain.difference(excluded), f)
} by {
    difference_subset(domain, excluded)
    bounded_on_subset(domain.difference(excluded), domain, f)
    is_bounded_on(domain.difference(excluded), f)
}

/// A maximum on a set remains a maximum on a difference when its point belongs to the difference.
theorem attains_maximum_at_difference[Domain, Value: PartialOrder](
    domain: Set[Domain], excluded: Set[Domain], f: Domain -> Value, point: Domain
) {
    domain.difference(excluded).contains(point) and attains_maximum_at(domain, f, point)
    implies attains_maximum_at(domain.difference(excluded), f, point)
} by {
    difference_subset(domain, excluded)
    attains_maximum_at_subset(domain.difference(excluded), domain, f, point)
    attains_maximum_at(domain.difference(excluded), f, point)
}

/// A minimum on a set remains a minimum on a difference when its point belongs to the difference.
theorem attains_minimum_at_difference[Domain, Value: PartialOrder](
    domain: Set[Domain], excluded: Set[Domain], f: Domain -> Value, point: Domain
) {
    domain.difference(excluded).contains(point) and attains_minimum_at(domain, f, point)
    implies attains_minimum_at(domain.difference(excluded), f, point)
} by {
    difference_subset(domain, excluded)
    attains_minimum_at_subset(domain.difference(excluded), domain, f, point)
    attains_minimum_at(domain.difference(excluded), f, point)
}

/// A common upper bound is an upper bound on the left domain.
theorem common_upper_bound_on_left[Domain, Value: PartialOrder](
    left: Set[Domain], right: Set[Domain], f: Domain -> Value, bound: Value
) {
    is_common_upper_bound_on(left, right, f, bound) implies is_upper_bound_on(left, f, bound)
} by {
    is_upper_bound_on(left, f, bound)
}

/// A common upper bound is an upper bound on the right domain.
theorem common_upper_bound_on_right[Domain, Value: PartialOrder](
    left: Set[Domain], right: Set[Domain], f: Domain -> Value, bound: Value
) {
    is_common_upper_bound_on(left, right, f, bound) implies is_upper_bound_on(right, f, bound)
} by {
    is_upper_bound_on(right, f, bound)
}

/// A common lower bound is a lower bound on the left domain.
theorem common_lower_bound_on_left[Domain, Value: PartialOrder](
    left: Set[Domain], right: Set[Domain], f: Domain -> Value, bound: Value
) {
    is_common_lower_bound_on(left, right, f, bound) implies is_lower_bound_on(left, f, bound)
} by {
    is_lower_bound_on(left, f, bound)
}

/// A common lower bound is a lower bound on the right domain.
theorem common_lower_bound_on_right[Domain, Value: PartialOrder](
    left: Set[Domain], right: Set[Domain], f: Domain -> Value, bound: Value
) {
    is_common_lower_bound_on(left, right, f, bound) implies is_lower_bound_on(right, f, bound)
} by {
    is_lower_bound_on(right, f, bound)
}

/// Upper bounds on both domains form a common upper bound.
theorem common_upper_bound_on_intro[Domain, Value: PartialOrder](
    left: Set[Domain], right: Set[Domain], f: Domain -> Value, bound: Value
) {
    is_upper_bound_on(left, f, bound) and is_upper_bound_on(right, f, bound)
    implies is_common_upper_bound_on(left, right, f, bound)
} by {
    is_common_upper_bound_on(left, right, f, bound)
}

/// Lower bounds on both domains form a common lower bound.
theorem common_lower_bound_on_intro[Domain, Value: PartialOrder](
    left: Set[Domain], right: Set[Domain], f: Domain -> Value, bound: Value
) {
    is_lower_bound_on(left, f, bound) and is_lower_bound_on(right, f, bound)
    implies is_common_lower_bound_on(left, right, f, bound)
} by {
    is_common_lower_bound_on(left, right, f, bound)
}

/// A common upper bound on two domains is an upper bound on their union.
theorem upper_bound_on_union_of_common[Domain, Value: PartialOrder](
    left: Set[Domain], right: Set[Domain], f: Domain -> Value, bound: Value
) {
    is_common_upper_bound_on(left, right, f, bound)
    implies is_upper_bound_on(left.union(right), f, bound)
} by {
    forall(point: Domain) {
        if left.union(right).contains(point) {
            if left.contains(point) {
                common_upper_bound_on_left(left, right, f, bound)
                upper_bound_on_apply(left, f, bound, point)
                f(point) <= bound
            } else {
                right.contains(point)
                common_upper_bound_on_right(left, right, f, bound)
                upper_bound_on_apply(right, f, bound, point)
                f(point) <= bound
            }
        }
    }
}

/// A common lower bound on two domains is a lower bound on their union.
theorem lower_bound_on_union_of_common[Domain, Value: PartialOrder](
    left: Set[Domain], right: Set[Domain], f: Domain -> Value, bound: Value
) {
    is_common_lower_bound_on(left, right, f, bound)
    implies is_lower_bound_on(left.union(right), f, bound)
} by {
    forall(point: Domain) {
        if left.union(right).contains(point) {
            if left.contains(point) {
                common_lower_bound_on_left(left, right, f, bound)
                lower_bound_on_apply(left, f, bound, point)
                bound <= f(point)
            } else {
                right.contains(point)
                common_lower_bound_on_right(left, right, f, bound)
                lower_bound_on_apply(right, f, bound, point)
                bound <= f(point)
            }
        }
    }
}

/// An upper bound on a union is a common upper bound on the two domains.
theorem common_upper_bound_on_of_union[Domain, Value: PartialOrder](
    left: Set[Domain], right: Set[Domain], f: Domain -> Value, bound: Value
) {
    is_upper_bound_on(left.union(right), f, bound)
    implies is_common_upper_bound_on(left, right, f, bound)
} by {
    sets_subset_union(left, right)
    left.subset(left.union(right))
    right.subset(left.union(right))
    upper_bound_on_subset(left, left.union(right), f, bound)
    upper_bound_on_subset(right, left.union(right), f, bound)
    is_upper_bound_on(left, f, bound)
    is_upper_bound_on(right, f, bound)
    is_common_upper_bound_on(left, right, f, bound)
}

/// A lower bound on a union is a common lower bound on the two domains.
theorem common_lower_bound_on_of_union[Domain, Value: PartialOrder](
    left: Set[Domain], right: Set[Domain], f: Domain -> Value, bound: Value
) {
    is_lower_bound_on(left.union(right), f, bound)
    implies is_common_lower_bound_on(left, right, f, bound)
} by {
    sets_subset_union(left, right)
    left.subset(left.union(right))
    right.subset(left.union(right))
    lower_bound_on_subset(left, left.union(right), f, bound)
    lower_bound_on_subset(right, left.union(right), f, bound)
    is_lower_bound_on(left, f, bound)
    is_lower_bound_on(right, f, bound)
    is_common_lower_bound_on(left, right, f, bound)
}

/// Upper bounds on a union are exactly common upper bounds on both domains.
theorem upper_bound_on_union_iff_common[Domain, Value: PartialOrder](
    left: Set[Domain], right: Set[Domain], f: Domain -> Value, bound: Value
) {
    is_upper_bound_on(left.union(right), f, bound) iff is_common_upper_bound_on(left, right, f, bound)
} by {
    if is_upper_bound_on(left.union(right), f, bound) {
        common_upper_bound_on_of_union(left, right, f, bound)
    }
    if is_common_upper_bound_on(left, right, f, bound) {
        upper_bound_on_union_of_common(left, right, f, bound)
    }
}

/// Lower bounds on a union are exactly common lower bounds on both domains.
theorem lower_bound_on_union_iff_common[Domain, Value: PartialOrder](
    left: Set[Domain], right: Set[Domain], f: Domain -> Value, bound: Value
) {
    is_lower_bound_on(left.union(right), f, bound) iff is_common_lower_bound_on(left, right, f, bound)
} by {
    if is_lower_bound_on(left.union(right), f, bound) {
        common_lower_bound_on_of_union(left, right, f, bound)
    }
    if is_common_lower_bound_on(left, right, f, bound) {
        lower_bound_on_union_of_common(left, right, f, bound)
    }
}

/// A common upper bound gives boundedness above on the union.
theorem bounded_above_on_union_of_common[Domain, Value: PartialOrder](
    left: Set[Domain], right: Set[Domain], f: Domain -> Value
) {
    is_common_bounded_above_on(left, right, f) implies is_bounded_above_on(left.union(right), f)
} by {
    let bound: Value satisfy {
        is_common_upper_bound_on(left, right, f, bound)
    }
    upper_bound_on_union_of_common(left, right, f, bound)
    is_upper_bound_on(left.union(right), f, bound)
    exists(witness: Value) {
        is_upper_bound_on(left.union(right), f, witness)
    }
    is_bounded_above_on(left.union(right), f)
}

/// A common lower bound gives boundedness below on the union.
theorem bounded_below_on_union_of_common[Domain, Value: PartialOrder](
    left: Set[Domain], right: Set[Domain], f: Domain -> Value
) {
    is_common_bounded_below_on(left, right, f) implies is_bounded_below_on(left.union(right), f)
} by {
    let bound: Value satisfy {
        is_common_lower_bound_on(left, right, f, bound)
    }
    lower_bound_on_union_of_common(left, right, f, bound)
    is_lower_bound_on(left.union(right), f, bound)
    exists(witness: Value) {
        is_lower_bound_on(left.union(right), f, witness)
    }
    is_bounded_below_on(left.union(right), f)
}

/// Common boundedness gives boundedness on the union.
theorem bounded_on_union_of_common[Domain, Value: PartialOrder](
    left: Set[Domain], right: Set[Domain], f: Domain -> Value
) {
    is_common_bounded_on(left, right, f) implies is_bounded_on(left.union(right), f)
} by {
    bounded_above_on_union_of_common(left, right, f)
    bounded_below_on_union_of_common(left, right, f)
    is_bounded_above_on(left.union(right), f)
    is_bounded_below_on(left.union(right), f)
    is_bounded_on(left.union(right), f)
}

/// Boundedness above on a union gives common boundedness above on the two domains.
theorem common_bounded_above_on_of_union[Domain, Value: PartialOrder](
    left: Set[Domain], right: Set[Domain], f: Domain -> Value
) {
    is_bounded_above_on(left.union(right), f) implies is_common_bounded_above_on(left, right, f)
} by {
    let bound: Value satisfy {
        is_upper_bound_on(left.union(right), f, bound)
    }
    common_upper_bound_on_of_union(left, right, f, bound)
    is_common_upper_bound_on(left, right, f, bound)
    exists(witness: Value) {
        is_common_upper_bound_on(left, right, f, witness)
    }
    is_common_bounded_above_on(left, right, f)
}

/// Boundedness below on a union gives common boundedness below on the two domains.
theorem common_bounded_below_on_of_union[Domain, Value: PartialOrder](
    left: Set[Domain], right: Set[Domain], f: Domain -> Value
) {
    is_bounded_below_on(left.union(right), f) implies is_common_bounded_below_on(left, right, f)
} by {
    let bound: Value satisfy {
        is_lower_bound_on(left.union(right), f, bound)
    }
    common_lower_bound_on_of_union(left, right, f, bound)
    is_common_lower_bound_on(left, right, f, bound)
    exists(witness: Value) {
        is_common_lower_bound_on(left, right, f, witness)
    }
    is_common_bounded_below_on(left, right, f)
}

/// Boundedness above on a union is exactly common boundedness above on both domains.
theorem bounded_above_on_union_iff_common[Domain, Value: PartialOrder](
    left: Set[Domain], right: Set[Domain], f: Domain -> Value
) {
    is_bounded_above_on(left.union(right), f) iff is_common_bounded_above_on(left, right, f)
} by {
    if is_bounded_above_on(left.union(right), f) {
        common_bounded_above_on_of_union(left, right, f)
    }
    if is_common_bounded_above_on(left, right, f) {
        bounded_above_on_union_of_common(left, right, f)
    }
}

/// Boundedness below on a union is exactly common boundedness below on both domains.
theorem bounded_below_on_union_iff_common[Domain, Value: PartialOrder](
    left: Set[Domain], right: Set[Domain], f: Domain -> Value
) {
    is_bounded_below_on(left.union(right), f) iff is_common_bounded_below_on(left, right, f)
} by {
    if is_bounded_below_on(left.union(right), f) {
        common_bounded_below_on_of_union(left, right, f)
    }
    if is_common_bounded_below_on(left, right, f) {
        bounded_below_on_union_of_common(left, right, f)
    }
}

/// Boundedness on a union gives common boundedness on the two domains.
theorem common_bounded_on_of_union[Domain, Value: PartialOrder](
    left: Set[Domain], right: Set[Domain], f: Domain -> Value
) {
    is_bounded_on(left.union(right), f) implies is_common_bounded_on(left, right, f)
} by {
    common_bounded_above_on_of_union(left, right, f)
    common_bounded_below_on_of_union(left, right, f)
    is_common_bounded_above_on(left, right, f)
    is_common_bounded_below_on(left, right, f)
    is_common_bounded_on(left, right, f)
}

/// A maximum on the left domain extends to the union when its value bounds the right domain above.
theorem attains_maximum_at_union_left[Domain, Value: PartialOrder](
    left: Set[Domain], right: Set[Domain], f: Domain -> Value, point: Domain
) {
    attains_maximum_at(left, f, point) and is_upper_bound_on(right, f, f(point))
    implies attains_maximum_at(left.union(right), f, point)
} by {
    maximum_value_is_upper_bound_on(left, f, point)
    is_upper_bound_on(left, f, f(point))
    common_upper_bound_on_intro(left, right, f, f(point))
    upper_bound_on_union_of_common(left, right, f, f(point))
    sets_subset_union(left, right)
    left.subset(left.union(right))
    left.contains(point)
    left.union(right).contains(point)
    is_upper_bound_on(left.union(right), f, f(point))
    attains_maximum_at(left.union(right), f, point)
}

/// A maximum on the right domain extends to the union when its value bounds the left domain above.
theorem attains_maximum_at_union_right[Domain, Value: PartialOrder](
    left: Set[Domain], right: Set[Domain], f: Domain -> Value, point: Domain
) {
    attains_maximum_at(right, f, point) and is_upper_bound_on(left, f, f(point))
    implies attains_maximum_at(left.union(right), f, point)
} by {
    maximum_value_is_upper_bound_on(right, f, point)
    is_upper_bound_on(right, f, f(point))
    common_upper_bound_on_intro(left, right, f, f(point))
    upper_bound_on_union_of_common(left, right, f, f(point))
    sets_subset_union(left, right)
    right.subset(left.union(right))
    right.contains(point)
    left.union(right).contains(point)
    is_upper_bound_on(left.union(right), f, f(point))
    attains_maximum_at(left.union(right), f, point)
}

/// A minimum on the left domain extends to the union when its value bounds the right domain below.
theorem attains_minimum_at_union_left[Domain, Value: PartialOrder](
    left: Set[Domain], right: Set[Domain], f: Domain -> Value, point: Domain
) {
    attains_minimum_at(left, f, point) and is_lower_bound_on(right, f, f(point))
    implies attains_minimum_at(left.union(right), f, point)
} by {
    minimum_value_is_lower_bound_on(left, f, point)
    is_lower_bound_on(left, f, f(point))
    common_lower_bound_on_intro(left, right, f, f(point))
    lower_bound_on_union_of_common(left, right, f, f(point))
    sets_subset_union(left, right)
    left.subset(left.union(right))
    left.contains(point)
    left.union(right).contains(point)
    is_lower_bound_on(left.union(right), f, f(point))
    attains_minimum_at(left.union(right), f, point)
}

/// A minimum on the right domain extends to the union when its value bounds the left domain below.
theorem attains_minimum_at_union_right[Domain, Value: PartialOrder](
    left: Set[Domain], right: Set[Domain], f: Domain -> Value, point: Domain
) {
    attains_minimum_at(right, f, point) and is_lower_bound_on(left, f, f(point))
    implies attains_minimum_at(left.union(right), f, point)
} by {
    minimum_value_is_lower_bound_on(right, f, point)
    is_lower_bound_on(right, f, f(point))
    common_lower_bound_on_intro(left, right, f, f(point))
    lower_bound_on_union_of_common(left, right, f, f(point))
    sets_subset_union(left, right)
    right.subset(left.union(right))
    right.contains(point)
    left.union(right).contains(point)
    is_lower_bound_on(left.union(right), f, f(point))
    attains_minimum_at(left.union(right), f, point)
}
