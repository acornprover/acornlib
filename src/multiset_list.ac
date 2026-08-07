from finite_set import FiniteSet, fs_from_list
from data.basic.logic import exists_intro
from list import List, list_contains_implies_count_geq_one,
    list_not_contains_impl_count_zero, count_append, is_permutation
from data.list.list_filter_count import filter_count_of_true, filter_count_of_false
from multiset import Multiset, multiset_ext, from_list_multiplicity, filter_multiplicity_true,
    filter_multiplicity_false, filter_contains_iff
from nat import Nat
from data.basic.set import list_set, list_set_contains_eq
numerals Nat

/// A cons list has one more copy of its head than its tail.
theorem list_count_cons_self[T](head: T, tail: List[T]) {
    List.cons(head, tail).count(head) = Nat.1 + tail.count(head)
} by {
}

/// A cons list has unchanged count away from its head.
theorem list_count_cons_other[T](head: T, tail: List[T], item: T) {
    head != item implies List.cons(head, tail).count(item) = tail.count(item)
} by {
    if head != item {
        List.cons(head, tail).count(item) = tail.count(item)
    }
}

/// Consing an element onto a list inserts one copy into its multiset.
theorem from_list_cons[T](head: T, tail: List[T]) {
    Multiset.from_list(List.cons(head, tail)) = Multiset.from_list(tail).insert(head)
} by {
    forall(x: T) {
        if head = x {
            list_count_cons_self(head, tail)
            Nat.1 + tail.count(x) = tail.count(x).suc
            Multiset.from_list(tail).insert(head).multiplicity(x) =
                Multiset.from_list(tail).multiplicity(x).suc
            Multiset.from_list(List.cons(head, tail)).multiplicity(x) =
                Multiset.from_list(tail).insert(head).multiplicity(x)
        } else {
            list_count_cons_other(head, tail, x)
            Multiset.from_list(tail).insert(head).multiplicity(x) =
                Multiset.from_list(tail).multiplicity(x)
            Multiset.from_list(List.cons(head, tail)).multiplicity(x) =
                Multiset.from_list(tail).insert(head).multiplicity(x)
        }
    }
    multiset_ext(Multiset.from_list(List.cons(head, tail)), Multiset.from_list(tail).insert(head))
}

/// A singleton list represents the singleton multiset.
theorem from_list_singleton[T](item: T) {
    Multiset.from_list(List.singleton(item)) = Multiset.singleton(item)
} by {
    forall(x: T) {
        if item = x {
            list_count_cons_self(item, List.nil[T])
            List.singleton(item).count(x) = Nat.1
            Multiset.singleton(item).multiplicity(x) = Nat.1
            Multiset.from_list(List.singleton(item)).multiplicity(x) = Multiset.singleton(item).multiplicity(x)
        } else {
            list_count_cons_other(item, List.nil[T], x)
            List.nil[T].count(x) = Nat.0
            Multiset.singleton(item).multiplicity(x) = Nat.0
            Multiset.from_list(List.singleton(item)).multiplicity(x) = Multiset.singleton(item).multiplicity(x)
        }
    }
    multiset_ext(Multiset.from_list(List.singleton(item)), Multiset.singleton(item))
}

/// List concatenation corresponds to pointwise multiset addition.
theorem from_list_append[T](left: List[T], right: List[T]) {
    Multiset.from_list(left + right) = Multiset.from_list(left) + Multiset.from_list(right)
} by {
    forall(x: T) {
        count_append(left, right, x)
        Multiset.from_list(left + right).multiplicity(x) =
            (Multiset.from_list(left) + Multiset.from_list(right)).multiplicity(x)
    }
    multiset_ext(Multiset.from_list(left + right), Multiset.from_list(left) + Multiset.from_list(right))
}

/// Filtering a list and then taking its multiset is the same as filtering the associated multiset.
theorem from_list_filter[T](items: List[T], pred: T -> Bool) {
    Multiset.from_list(items.filter(pred)) = Multiset.from_list(items).filter(pred)
} by {
    forall(x: T) {
        if pred(x) {
            filter_count_of_true[T](items, pred, x)
            filter_multiplicity_true[T](Multiset.from_list(items), pred, x)
            Multiset.from_list(items.filter(pred)).multiplicity(x) =
                Multiset.from_list(items).filter(pred).multiplicity(x)
        }
        if not pred(x) {
            filter_count_of_false[T](items, pred, x)
            filter_multiplicity_false[T](Multiset.from_list(items), pred, x)
            Multiset.from_list(items.filter(pred)).multiplicity(x) =
                Multiset.from_list(items).filter(pred).multiplicity(x)
        }
        Multiset.from_list(items.filter(pred)).multiplicity(x) =
            Multiset.from_list(items).filter(pred).multiplicity(x)
    }
    multiset_ext(Multiset.from_list(items.filter(pred)), Multiset.from_list(items).filter(pred))
}

/// A list contains an element exactly when its associated multiset contains it.
theorem from_list_contains[T](items: List[T], item: T) {
    Multiset.from_list(items).contains(item) = items.contains(item)
} by {
    if items.contains(item) {
        list_contains_implies_count_geq_one(items, item)
        items.count(item) >= Nat.1
        Multiset.from_list(items).multiplicity(item) = items.count(item)
        Multiset.from_list(items).multiplicity(item) >= Nat.1
        if Multiset.from_list(items).multiplicity(item) = Nat.0 {
            Nat.1 <= Nat.0
            false
        }
        Multiset.from_list(items).contains(item)
    }
    if Multiset.from_list(items).contains(item) {
        if not items.contains(item) {
            list_not_contains_impl_count_zero(items, item)
            false
        }
    }
}

/// Containment in a filtered list multiset is list containment together with the predicate.
theorem from_list_filter_contains[T](items: List[T], pred: T -> Bool, item: T) {
    Multiset.from_list(items.filter(pred)).contains(item) = (items.contains(item) and pred(item))
} by {
    from_list_filter(items, pred)
    filter_contains_iff(Multiset.from_list(items), pred, item)
    from_list_contains(items, item)
}

/// Permutation-equivalent lists determine the same multiset.
theorem permutation_imp_from_list_eq[T](a: List[T], b: List[T]) {
    is_permutation(a, b) implies Multiset.from_list(a) = Multiset.from_list(b)
} by {
    if is_permutation(a, b) {
        forall(x: T) {
            from_list_multiplicity(a, x)
            a.count(x) = b.count(x)
            from_list_multiplicity(b, x)
            Multiset.from_list(a).multiplicity(x) = Multiset.from_list(b).multiplicity(x)
        }
        forall(x: T) { Multiset.from_list(a).multiplicity(x) = Multiset.from_list(b).multiplicity(x) }
        multiset_ext(Multiset.from_list(a), Multiset.from_list(b))
        Multiset.from_list(a) = Multiset.from_list(b)
    }
}

/// Equal list multisets have equal counts, hence the lists are permutations.
theorem from_list_eq_imp_permutation[T](a: List[T], b: List[T]) {
    Multiset.from_list(a) = Multiset.from_list(b) implies is_permutation(a, b)
} by {
    if Multiset.from_list(a) = Multiset.from_list(b) {
        forall(x: T) {
            a.count(x) = b.count(x)
        }
    }
}

/// Equality of associated multisets is equivalent to list permutation.
theorem from_list_eq_iff_permutation[T](a: List[T], b: List[T]) {
    (Multiset.from_list(a) = Multiset.from_list(b)) = is_permutation(a, b)
} by {
    permutation_imp_from_list_eq(a, b)
    from_list_eq_imp_permutation(a, b)
}

/// The chosen finite support for `from_list` contains exactly the elements of the list.
theorem from_list_support_contains_list[T](items: List[T], item: T) {
    FiniteSet.from_list(items).contains(item) = items.contains(item)
} by {
    fs_from_list(items).underlying_set = list_set(items)
    list_set_contains_eq(items, item)
}

/// The chosen finite support for `from_list` contains exactly the elements of the multiset.
theorem from_list_support_contains[T](items: List[T], item: T) {
    FiniteSet.from_list(items).contains(item) = Multiset.from_list(items).contains(item)
} by {
    from_list_support_contains_list(items, item)
    from_list_contains(items, item)
}

/// Outside the finite set of elements occurring in a list, the associated multiplicity is zero.
theorem from_list_multiplicity_zero_of_not_in_support[T](items: List[T], item: T) {
    not FiniteSet.from_list(items).contains(item) implies
    Multiset.from_list(items).multiplicity(item) = Nat.0
} by {
    if not FiniteSet.from_list(items).contains(item) {
        from_list_support_contains_list(items, item)
        list_not_contains_impl_count_zero(items, item)
        Multiset.from_list(items).multiplicity(item) = Nat.0
    }
}

/// The set of elements occurring in a list supports the associated multiset.
theorem from_list_supported_by_list_elements[T](items: List[T]) {
    forall(x: T) {
        not FiniteSet.from_list(items).contains(x) implies
        Multiset.from_list(items).multiplicity(x) = Nat.0
    }
} by {
    forall(x: T) {
        if not FiniteSet.from_list(items).contains(x) {
            from_list_multiplicity_zero_of_not_in_support(items, x)
            Multiset.from_list(items).multiplicity(x) = Nat.0
        }
    }
}

/// Multisets obtained from lists have an explicit finite support witness.
theorem from_list_has_finite_support_witness[T](items: List[T]) {
    exists(support: FiniteSet[T]) {
        forall(x: T) {
            not support.contains(x) implies Multiset.from_list(items).multiplicity(x) = Nat.0
        }
    }
} by {
    from_list_supported_by_list_elements(items)
    exists_intro(function(support: FiniteSet[T]) {
        forall(x: T) {
            not support.contains(x) implies Multiset.from_list(items).multiplicity(x) = Nat.0
        }
    }, FiniteSet.from_list(items))
}
