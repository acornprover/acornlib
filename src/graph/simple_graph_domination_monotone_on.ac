from nat import Nat
from finite_set import FiniteSet, finite_set_subset_contains
from data.finite.finite_set_card import fs_card
from graph.simple_graph import SimpleGraph
from graph.simple_graph_domination import is_dominating_set, is_dominating_set_apply,
    is_dominating_set_intro, is_dominated, is_dominated_of_contains, is_dominated_of_adj,
    has_neighbor_in, has_neighbor_in_witness
from graph.simple_graph_domination_number import domination_number, domination_number_is_least
from graph.simple_graph_minimum_dominating import is_minimum_dominating_set,
    is_minimum_dominating_set_apply, is_dominating_subset, is_dominating_subset_apply,
    minimum_dominating_set_exists

numerals Nat

/// True if every edge of `g` between vertices of `s` is also an edge of `h`.
///
/// The version of `has_all_edges_of` restricted to a vertex set. The unrestricted relation is
/// too strong for graphs that agree only on part of the vertex type: the cycle of length `n`
/// has every edge the path has between vertices below `n`, but not the edge from `n - 1` to
/// `n`, which lies outside its range.
define has_all_edges_of_on[V](h: SimpleGraph[V], g: SimpleGraph[V], s: FiniteSet[V]) -> Bool {
    forall(x: V, y: V) {
        s.contains(x) and s.contains(y) and g.adj(x, y) implies h.adj(x, y)
    }
}

/// An edge of the smaller graph inside the vertex set is an edge of the larger.
theorem has_all_edges_of_on_apply[V](
    h: SimpleGraph[V], g: SimpleGraph[V], s: FiniteSet[V], x: V, y: V
) {
    has_all_edges_of_on(h, g, s) and s.contains(x) and s.contains(y) and g.adj(x, y)
        implies h.adj(x, y)
} by {
    if has_all_edges_of_on(h, g, s) and s.contains(x) and s.contains(y) and g.adj(x, y) {
        (has_all_edges_of_on(h, g, s) = forall(a: V, b: V) {
            s.contains(a) and s.contains(b) and g.adj(a, b) implies h.adj(a, b)
        })
        (s.contains(x) and s.contains(y) and g.adj(x, y) implies h.adj(x, y))
        h.adj(x, y)
    }
}

/// A pointwise edge containment inside the vertex set gives the relation.
theorem has_all_edges_of_on_intro[V](
    h: SimpleGraph[V], g: SimpleGraph[V], s: FiniteSet[V]
) {
    (forall(x: V, y: V) {
        s.contains(x) and s.contains(y) and g.adj(x, y) implies h.adj(x, y)
    }) implies has_all_edges_of_on(h, g, s)
} by {
    if forall(x: V, y: V) {
        s.contains(x) and s.contains(y) and g.adj(x, y) implies h.adj(x, y)
    } {
        (has_all_edges_of_on(h, g, s) = forall(a: V, b: V) {
            s.contains(a) and s.contains(b) and g.adj(a, b) implies h.adj(a, b)
        })
        has_all_edges_of_on(h, g, s)
    }
}

/// Adding edges inside the vertex set preserves domination of a vertex of that set.
///
/// The dominating set has to lie inside the vertex set as well, since the witness that makes a
/// vertex dominated is one of its members and the edge to it is only guaranteed there.
theorem is_dominated_of_has_all_edges_on[V](
    h: SimpleGraph[V], g: SimpleGraph[V], s: FiniteSet[V], d: FiniteSet[V], v: V
) {
    has_all_edges_of_on(h, g, s) and d.subset_eq(s) and s.contains(v)
        and is_dominated(g, d, v) implies is_dominated(h, d, v)
} by {
    if has_all_edges_of_on(h, g, s) and d.subset_eq(s) and s.contains(v)
        and is_dominated(g, d, v) {
        if d.contains(v) {
            is_dominated_of_contains(h, d, v)
            is_dominated(h, d, v)
        }
        if not d.contains(v) {
            has_neighbor_in(g, d, v)
            has_neighbor_in_witness(g, d, v)
            let (w: V) satisfy {
                d.contains(w) and g.adj(w, v)
            }
            finite_set_subset_contains(d, s, w)
            s.contains(w)
            has_all_edges_of_on_apply(h, g, s, w, v)
            h.adj(w, v)
            is_dominated_of_adj(h, d, v, w)
            is_dominated(h, d, v)
        }
        is_dominated(h, d, v)
    }
}

/// A dominating subset stays dominating when edges are added inside the vertex set.
theorem is_dominating_set_of_has_all_edges_on[V](
    h: SimpleGraph[V], g: SimpleGraph[V], s: FiniteSet[V], d: FiniteSet[V]
) {
    has_all_edges_of_on(h, g, s) and d.subset_eq(s) and is_dominating_set(g, s, d)
        implies is_dominating_set(h, s, d)
} by {
    if has_all_edges_of_on(h, g, s) and d.subset_eq(s) and is_dominating_set(g, s, d) {
        forall(v: V) {
            if s.contains(v) {
                is_dominating_set_apply(g, s, d, v)
                is_dominated(g, d, v)
                is_dominated_of_has_all_edges_on(h, g, s, d, v)
                is_dominated(h, d, v)
            }
            (s.contains(v) implies is_dominated(h, d, v))
        }
        is_dominating_set_intro(h, s, d)
        is_dominating_set(h, s, d)
    }
}

/// Adding edges inside the vertex set cannot raise the domination number over that set.
///
/// The domination number is defined through dominating *subsets* of the vertex set, so the
/// minimum dominating set of the sparser graph already lies where the added edges are, and it
/// dominates the denser graph there.
theorem domination_number_antitone_in_edges_on[V](
    h: SimpleGraph[V], g: SimpleGraph[V], s: FiniteSet[V]
) {
    has_all_edges_of_on(h, g, s) implies domination_number(h, s) <= domination_number(g, s)
} by {
    if has_all_edges_of_on(h, g, s) {
        minimum_dominating_set_exists(g, s)
        let (d: FiniteSet[V]) satisfy {
            is_minimum_dominating_set(g, s, d)
        }
        is_minimum_dominating_set_apply(g, s, d)
        is_dominating_subset(g, s, d)
        fs_card(d) = domination_number(g, s)
        is_dominating_subset_apply(g, s, d)
        d.subset_eq(s)
        is_dominating_set(g, s, d)
        is_dominating_set_of_has_all_edges_on(h, g, s, d)
        is_dominating_set(h, s, d)
        domination_number_is_least(h, s, d)
        domination_number(h, s) <= fs_card(d)
        domination_number(h, s) <= domination_number(g, s)
    }
}
