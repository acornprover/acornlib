from nat import Nat
from data.fin.fin import Fin
from data.basic.relation_basic import is_symmetric, is_irreflexive
from graph.simple_graph import SimpleGraph

numerals Nat

/// Adjacency on the four-vertex diamond: every pair but `2` and `3`.
///
/// The diamond is the complete graph on four vertices with one edge removed. Vertices `0` and
/// `1` are the ends of the remaining edge between the two sides, and `2` and `3` are the two
/// vertices the missing edge would have joined.
///
/// Distinctness is stated on the labels rather than on the vertices, which keeps every step
/// arithmetic; the two readings agree because a claw vertex is determined by its label.
define diamond4_adj(x: Fin[Nat.4], y: Fin[Nat.4]) -> Bool {
    x.value != y.value
        and (x.value = Nat.0 or x.value = Nat.1 or y.value = Nat.0 or y.value = Nat.1)
}

/// Diamond adjacency is symmetric.
theorem diamond4_adj_is_symmetric {
    is_symmetric(diamond4_adj)
} by {
    forall(x: Fin[Nat.4], y: Fin[Nat.4]) {
        if diamond4_adj(x, y) {
            (diamond4_adj(x, y) = (x.value != y.value
                and (x.value = Nat.0 or x.value = Nat.1
                    or y.value = Nat.0 or y.value = Nat.1)))
            (diamond4_adj(y, x) = (y.value != x.value
                and (y.value = Nat.0 or y.value = Nat.1
                    or x.value = Nat.0 or x.value = Nat.1)))
            diamond4_adj(y, x)
        }
        (diamond4_adj(x, y) implies diamond4_adj(y, x))
    }
    (is_symmetric(diamond4_adj) = forall(x: Fin[Nat.4], y: Fin[Nat.4]) {
        diamond4_adj(x, y) implies diamond4_adj(y, x)
    })
    is_symmetric(diamond4_adj)
}

/// No vertex of the diamond is adjacent to itself.
theorem diamond4_adj_is_irreflexive {
    is_irreflexive(diamond4_adj)
} by {
    forall(x: Fin[Nat.4]) {
        if diamond4_adj(x, x) {
            (diamond4_adj(x, x) = (x.value != x.value
                and (x.value = Nat.0 or x.value = Nat.1
                    or x.value = Nat.0 or x.value = Nat.1)))
            x.value != x.value
            false
        }
        not diamond4_adj(x, x)
    }
    (is_irreflexive(diamond4_adj) = forall(x: Fin[Nat.4]) { not diamond4_adj(x, x) })
    is_irreflexive(diamond4_adj)
}

/// `diamond4_adj` satisfies the simple-graph constraint.
theorem diamond4_adj_constraint {
    SimpleGraph.constraint(diamond4_adj)
} by {
    diamond4_adj_is_symmetric
    diamond4_adj_is_irreflexive
    (SimpleGraph.constraint(diamond4_adj)
        = (is_symmetric(diamond4_adj) and is_irreflexive(diamond4_adj)))
}

/// The diamond exists as a simple graph.
theorem diamond4_graph_exists {
    exists(g: SimpleGraph[Fin[Nat.4]]) { SimpleGraph.new(diamond4_adj) = Option.some(g) }
} by {
    diamond4_adj_constraint
}

/// The diamond on four vertices.
let diamond4_graph: SimpleGraph[Fin[Nat.4]] satisfy {
    SimpleGraph.new(diamond4_adj) = Option.some(diamond4_graph)
}

/// Adjacency in the diamond graph is being distinct with one end on the remaining edge.
theorem diamond4_graph_adj_iff(x: Fin[Nat.4], y: Fin[Nat.4]) {
    diamond4_graph.adj(x, y) = (x.value != y.value
        and (x.value = Nat.0 or x.value = Nat.1 or y.value = Nat.0 or y.value = Nat.1))
} by {
    diamond4_adj_constraint
    SimpleGraph.new(diamond4_adj) = Option.some(diamond4_graph)
    diamond4_graph.adj = diamond4_adj
    diamond4_graph.adj(x, y) = diamond4_adj(x, y)
    (diamond4_adj(x, y) = (x.value != y.value
        and (x.value = Nat.0 or x.value = Nat.1 or y.value = Nat.0 or y.value = Nat.1)))
}
