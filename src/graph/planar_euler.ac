from nat import Nat, add_comm, add_imp_sub, suc_sub_one, mul_zero_right, mul_one_right,
    mul_comm, distrib_left, div_mul
from pair import Pair
from data.basic.logic import false_implies
from finite_set import FiniteSet, fs_insert, fs_remove, finite_set_sum, finite_set_sum_empty,
    finite_set_sum_insert, finite_set_ext_contains, finite_set_empty_contains_eq
from data.finite.finite_set_membership import fs_remove_contains_eq, fs_insert_contains_eq,
    fs_insert_contains_cases
from data.finite.finite_set_card import fs_card, fs_card_empty
from data.finite.finite_set_card_ops import fs_card_insert_of_not_contains, fs_card_remove_of_contains
from data.finite.finite_set_unique_induction import finite_set_strong_induction
from graph.simple_graph import SimpleGraph, complete_graph, complete_graph_adj_iff_ne,
    induced_subgraph, induced_subgraph_adj_of_base_adj
from graph.simple_graph_connectivity import simple_graph_reachable, simple_graph_reachable_refl,
    simple_graph_reachable_of_adj
from graph.simple_graph_degree import degree, neighborhood, degree_eq_neighborhood_card,
    complete_graph_neighborhood_contains_eq
from graph.simple_graph_degree_sum import degree_fn, degree_fn_eq, degree_sum, degree_sum_eq
from graph.simple_graph_edges import directed_edges, directed_edges_contains_eq,
    directed_edge_count, directed_edge_count_eq
from graph.simple_graph_edge_fibers import directed_edges_from_set, directed_edges_from_set_contains_eq
from graph.simple_graph_handshake import handshake_lemma
from graph.simple_graph_eulerian import finite_set_sum_pointwise_eq
from graph.simple_graph_tree import is_tree, is_acyclic, is_acyclic_of_card_le_two, simple_graph_connected_on

numerals Nat

/// True if the graph `g` admits a planar embedding: a drawing of `g` in the plane
/// with no two edges crossing.
///
/// PLACEHOLDER: a full formalization of planar embeddings (topological or
/// combinatorial) is out of scope here. Until a genuine embedding definition
/// replaces this body, the predicate is the constant `false`, so every theorem
/// stated under it is vacuous; the base cases below (trees, cycles, `K4`) are
/// proved directly, without this predicate.
define is_planar[V](g: SimpleGraph[V]) -> Bool {
    false
}

/// The number of (undirected) edges of `g` on the vertex set `s`.
///
/// Edges are counted through the ordered adjacent pairs: each undirected edge
/// appears once in each orientation, so the number of edges is half the number
/// of ordered adjacent pairs.
define edge_count[V](g: SimpleGraph[V], s: FiniteSet[V]) -> Nat {
    directed_edge_count(g, s).div(Nat.2)
}

/// The edge count of a graph equals half its directed edge count.
theorem edge_count_eq[V](g: SimpleGraph[V], s: FiniteSet[V]) {
    edge_count(g, s) = directed_edge_count(g, s).div(Nat.2)
} by {
}

/// The placeholder planarity predicate is never satisfied.
///
/// Once a genuine embedding definition replaces `is_planar`, this theorem becomes
/// the proof obligation that some graphs are planar, and the statements under
/// `is_planar` become real theorems.
theorem not_is_planar[V](g: SimpleGraph[V]) {
    not is_planar(g)
} by {
    if is_planar(g) {
        is_planar(g) = false
        false
    }
}

/// Euler's formula, general form: a connected planar graph with `V` vertices, `E`
/// edges and `F` faces satisfies `V + F = E + 2`.
///
/// The additive form is the integer equation `V - E + F = 2` rearranged so that no
/// truncated natural subtraction appears (`E` may exceed `V`, as in `K4`). The
/// number of faces is not yet defined as a region count, so the formula is stated
/// existentially. It is vacuous under the placeholder `is_planar`.
theorem euler_formula_general[V](g: SimpleGraph[V], s: FiniteSet[V]) {
    is_planar(g) and simple_graph_connected_on(g, s) implies
        exists(f: Nat) {
            fs_card(s) + f = edge_count(g, s) + Nat.2
        }
} by {
    if is_planar(g) and simple_graph_connected_on(g, s) {
        is_planar(g)
        is_planar(g) = false
        false
    }
    (is_planar(g) and simple_graph_connected_on(g, s)) = false
    false_implies(exists(f: Nat) {
        fs_card(s) + f = edge_count(g, s) + Nat.2
    })
    (false implies exists(f: Nat) {
        fs_card(s) + f = edge_count(g, s) + Nat.2
    }) = true
    ((is_planar(g) and simple_graph_connected_on(g, s)) implies
        exists(f: Nat) {
            fs_card(s) + f = edge_count(g, s) + Nat.2
        }) = true
    ((is_planar(g) and simple_graph_connected_on(g, s)) implies
        exists(f: Nat) {
            fs_card(s) + f = edge_count(g, s) + Nat.2
        })
}

// ------------------------------------------------------------
// Generic counting machinery
// ------------------------------------------------------------

/// The constant function on `V` with value `c`.
define nat_const_fn[V](c: Nat) -> (V -> Nat) {
    function(x: V) {
        c
    }
}

/// The sum of a constant function over `s` is the constant times the cardinality of `s`.
theorem finite_set_sum_const[V](s: FiniteSet[V], c: Nat) {
    finite_set_sum(s, nat_const_fn[V](c)) = c * fs_card(s)
} by {
    define p(t: FiniteSet[V]) -> Bool {
        finite_set_sum(t, nat_const_fn[V](c)) = c * fs_card(t)
    }

    finite_set_sum_empty(nat_const_fn[V](c))
    finite_set_sum(FiniteSet.empty[V], nat_const_fn[V](c)) = Nat.0
    mul_zero_right(c)
    c * Nat.0 = Nat.0
    finite_set_sum(FiniteSet.empty[V], nat_const_fn[V](c)) = c * fs_card(FiniteSet.empty[V])
    p(FiniteSet.empty[V])

    forall(t: FiniteSet[V], item: V) {
        if p(t) and not t.contains(item) {
            finite_set_sum_insert(t, item, nat_const_fn[V](c))
            finite_set_sum(fs_insert(t, item), nat_const_fn[V](c)) =
                nat_const_fn[V](c)(item) + finite_set_sum(t, nat_const_fn[V](c))
            nat_const_fn[V](c)(item) = c
            finite_set_sum(t, nat_const_fn[V](c)) = c * fs_card(t)
            finite_set_sum(fs_insert(t, item), nat_const_fn[V](c)) = c + c * fs_card(t)
            add_comm(c, c * fs_card(t))
            c + c * fs_card(t) = c * fs_card(t) + c
            distrib_left(c, fs_card(t), Nat.1)
            c * (fs_card(t) + Nat.1) = c * fs_card(t) + c * Nat.1
            mul_one_right(c)
            c * Nat.1 = c
            c * (fs_card(t) + Nat.1) = c * fs_card(t) + c
            c * fs_card(t) + c = c * (fs_card(t) + Nat.1)
            finite_set_sum(fs_insert(t, item), nat_const_fn[V](c)) = c * (fs_card(t) + Nat.1)
            fs_card_insert_of_not_contains(t, item)
            fs_card(fs_insert(t, item)) = fs_card(t) + Nat.1
            c * fs_card(fs_insert(t, item)) = c * (fs_card(t) + Nat.1)
            finite_set_sum(fs_insert(t, item), nat_const_fn[V](c)) = c * fs_card(fs_insert(t, item))
        }
        p(t) and not t.contains(item) implies p(fs_insert(t, item))
    }
    finite_set_strong_induction(p, s)
    p(s)
    finite_set_sum(s, nat_const_fn[V](c)) = c * fs_card(s)
}

/// The directed edges leaving the whole vertex set are exactly the directed edges.
theorem directed_edges_from_set_self_eq[V](g: SimpleGraph[V], s: FiniteSet[V]) {
    directed_edges_from_set(g, s, s) = directed_edges(g, s)
} by {
    forall(p: Pair[V, V]) {
        directed_edges_from_set_contains_eq(g, s, s, p)
        directed_edges_from_set(g, s, s).contains(p) =
            (directed_edges(g, s).contains(p) and s.contains(p.first))
        directed_edges_contains_eq(g, s, p)
        directed_edges(g, s).contains(p) =
            (s.contains(p.first) and s.contains(p.second) and g.adj(p.first, p.second))
        if directed_edges_from_set(g, s, s).contains(p) {
            directed_edges(g, s).contains(p)
        }
        if directed_edges(g, s).contains(p) {
            s.contains(p.first)
            directed_edges_from_set(g, s, s).contains(p)
        }
        directed_edges_from_set(g, s, s).contains(p) = directed_edges(g, s).contains(p)
    }
    finite_set_ext_contains(directed_edges_from_set(g, s, s), directed_edges(g, s))
}

/// The sum of the degrees equals the number of ordered adjacent pairs.
///
/// This is the handshake lemma phrased with the directed edge count: every directed
/// edge has its first component in `s`, so the set of edges leaving `s` is the
/// whole set of directed edges.
theorem degree_sum_eq_directed_edge_count[V](g: SimpleGraph[V], s: FiniteSet[V]) {
    degree_sum(g, s) = directed_edge_count(g, s)
} by {
    handshake_lemma(g, s)
    degree_sum(g, s) = fs_card(directed_edges_from_set(g, s, s))
    directed_edges_from_set_self_eq(g, s)
    directed_edges_from_set(g, s, s) = directed_edges(g, s)
    fs_card(directed_edges_from_set(g, s, s)) = fs_card(directed_edges(g, s))
    directed_edge_count_eq(g, s)
    directed_edge_count(g, s) = fs_card(directed_edges(g, s))
    degree_sum(g, s) = directed_edge_count(g, s)
}

/// In the complete graph, the neighborhood of a vertex of `s` is `s` with that vertex removed.
theorem complete_graph_neighborhood_eq_remove[V](s: FiniteSet[V], v: V) {
    s.contains(v) implies neighborhood(complete_graph[V], s, v) = fs_remove(s, v)
} by {
    if s.contains(v) {
        forall(w: V) {
            complete_graph_neighborhood_contains_eq(s, v, w)
            neighborhood(complete_graph[V], s, v).contains(w) = (s.contains(w) and v != w)
            fs_remove_contains_eq(s, v, w)
            fs_remove(s, v).contains(w) = (s.contains(w) and w != v)
            if neighborhood(complete_graph[V], s, v).contains(w) {
                s.contains(w)
                v != w
                w != v
                fs_remove_contains_eq(s, v, w)
                fs_remove(s, v).contains(w) = (s.contains(w) and w != v)
                fs_remove(s, v).contains(w)
            }
            if fs_remove(s, v).contains(w) {
                s.contains(w)
                w != v
                v != w
                complete_graph_neighborhood_contains_eq(s, v, w)
                neighborhood(complete_graph[V], s, v).contains(w) = (s.contains(w) and v != w)
                neighborhood(complete_graph[V], s, v).contains(w)
            }
            neighborhood(complete_graph[V], s, v).contains(w) = fs_remove(s, v).contains(w)
            neighborhood(complete_graph[V], s, v).contains(w) =
                neighborhood(complete_graph[V], s, v).underlying_set.contains(w)
            fs_remove(s, v).contains(w) = fs_remove(s, v).underlying_set.contains(w)
            neighborhood(complete_graph[V], s, v).underlying_set.contains(w) =
                fs_remove(s, v).underlying_set.contains(w)
        }
        finite_set_ext_contains(neighborhood(complete_graph[V], s, v), fs_remove(s, v))
        neighborhood(complete_graph[V], s, v) = fs_remove(s, v)
    }
}

/// In the complete graph every vertex of `s` has degree one less than the size of `s`.
theorem complete_graph_degree[V](s: FiniteSet[V], v: V) {
    s.contains(v) implies degree(complete_graph[V], s, v) = fs_card(s) - Nat.1
} by {
    if s.contains(v) {
        complete_graph_neighborhood_eq_remove(s, v)
        neighborhood(complete_graph[V], s, v) = fs_remove(s, v)
        degree_eq_neighborhood_card(complete_graph[V], s, v)
        degree(complete_graph[V], s, v) = fs_card(neighborhood(complete_graph[V], s, v))
        fs_card(neighborhood(complete_graph[V], s, v)) = fs_card(fs_remove(s, v))
        degree(complete_graph[V], s, v) = fs_card(fs_remove(s, v))
        fs_card_remove_of_contains(s, v)
        fs_card(s) = fs_card(fs_remove(s, v)) + Nat.1
        add_imp_sub(fs_card(fs_remove(s, v)), Nat.1, fs_card(s))
        fs_card(s) - Nat.1 = fs_card(fs_remove(s, v))
        degree(complete_graph[V], s, v) = fs_card(s) - Nat.1
    }
}

/// In the complete graph, the directed edge count on a vertex set of size `n` is `n (n - 1)`.
theorem complete_graph_directed_edge_count_of_card[V](s: FiniteSet[V], n: Nat) {
    fs_card(s) = n implies directed_edge_count(complete_graph[V], s) = (n - Nat.1) * n
} by {
    if fs_card(s) = n {
        degree_sum_eq_directed_edge_count(complete_graph[V], s)
        degree_sum(complete_graph[V], s) = directed_edge_count(complete_graph[V], s)
        degree_sum_eq(complete_graph[V], s)
        degree_sum(complete_graph[V], s) = finite_set_sum(s, degree_fn(complete_graph[V], s))
        forall(x: V) {
            if s.contains(x) {
                degree_fn_eq(complete_graph[V], s, x)
                degree_fn(complete_graph[V], s)(x) = degree(complete_graph[V], s, x)
                complete_graph_degree(s, x)
                degree(complete_graph[V], s, x) = fs_card(s) - Nat.1
                fs_card(s) - Nat.1 = n - Nat.1
                degree(complete_graph[V], s, x) = n - Nat.1
                nat_const_fn[V](n - Nat.1)(x) = n - Nat.1
                degree_fn(complete_graph[V], s)(x) = nat_const_fn[V](n - Nat.1)(x)
            }
            s.contains(x) implies degree_fn(complete_graph[V], s)(x) = nat_const_fn[V](n - Nat.1)(x)
        }
        finite_set_sum_pointwise_eq(s, degree_fn(complete_graph[V], s), nat_const_fn[V](n - Nat.1))
        finite_set_sum(s, degree_fn(complete_graph[V], s)) = finite_set_sum(s, nat_const_fn[V](n - Nat.1))
        finite_set_sum_const(s, n - Nat.1)
        finite_set_sum(s, nat_const_fn[V](n - Nat.1)) = (n - Nat.1) * fs_card(s)
        (n - Nat.1) * fs_card(s) = (n - Nat.1) * n
        finite_set_sum(s, degree_fn(complete_graph[V], s)) = (n - Nat.1) * n
        degree_sum(complete_graph[V], s) = (n - Nat.1) * n
        directed_edge_count(complete_graph[V], s) = (n - Nat.1) * n
    }
}

/// In the complete graph, the edge count on a vertex set of size `n` is `n (n - 1) / 2`.
theorem edge_count_complete_of_card[V](s: FiniteSet[V], n: Nat) {
    fs_card(s) = n implies edge_count(complete_graph[V], s) = ((n - Nat.1) * n).div(Nat.2)
} by {
    if fs_card(s) = n {
        edge_count_eq(complete_graph[V], s)
        edge_count(complete_graph[V], s) = directed_edge_count(complete_graph[V], s).div(Nat.2)
        complete_graph_directed_edge_count_of_card(s, n)
        directed_edge_count(complete_graph[V], s) = (n - Nat.1) * n
        directed_edge_count(complete_graph[V], s).div(Nat.2) = ((n - Nat.1) * n).div(Nat.2)
        edge_count(complete_graph[V], s) = ((n - Nat.1) * n).div(Nat.2)
    }
}

// ------------------------------------------------------------
// Base cases: tree, cycle, K4
//
// The concrete base cases use the naturals as the vertex type and finite vertex
// sets written out as literal insertions, so no ordering or witness machinery is
// needed. The complete graphs `K2`, `K3` and `K4` serve respectively as the tree
// on two vertices, the cycle `C3` (a triangle), and `K4` itself.
// ------------------------------------------------------------

/// The two-element vertex set `{0, 1}`.
let s2: FiniteSet[Nat] = fs_insert(fs_insert(FiniteSet.empty[Nat], Nat.0), Nat.1)

/// The three-element vertex set `{0, 1, 2}`.
let s3: FiniteSet[Nat] = fs_insert(fs_insert(fs_insert(FiniteSet.empty[Nat], Nat.0), Nat.1), Nat.2)

/// The four-element vertex set `{0, 1, 2, 3}`.
let s4: FiniteSet[Nat] = fs_insert(fs_insert(fs_insert(fs_insert(FiniteSet.empty[Nat], Nat.0), Nat.1), Nat.2), Nat.3)

/// An element distinct from the inserted item and absent from the original set
/// stays absent after the insertion.
theorem fs_insert_not_contains_of_ne[V](s: FiniteSet[V], item: V, x: V) {
    x != item and not s.contains(x) implies not fs_insert(s, item).contains(x)
} by {
    if x != item and not s.contains(x) {
        if fs_insert(s, item).contains(x) {
            fs_insert_contains_cases(s, item, x)
            x = item or s.contains(x)
            if x = item {
                x != item
                false
            }
            if s.contains(x) {
                not s.contains(x)
                false
            }
            false
        }
        not fs_insert(s, item).contains(x)
    }
}

/// The two-element vertex set has two elements.
theorem s2_card {
    fs_card(s2) = Nat.2
} by {
    fs_card_insert_of_not_contains(FiniteSet.empty[Nat], Nat.0)
    not FiniteSet.empty[Nat].contains(Nat.0)
    fs_card(fs_insert(FiniteSet.empty[Nat], Nat.0)) = fs_card(FiniteSet.empty[Nat]) + Nat.1
    fs_card_empty[Nat]
    fs_card(FiniteSet.empty[Nat]) = Nat.0
    fs_card(fs_insert(FiniteSet.empty[Nat], Nat.0)) = Nat.1
    fs_card_insert_of_not_contains(fs_insert(FiniteSet.empty[Nat], Nat.0), Nat.1)
    fs_insert_contains_eq(FiniteSet.empty[Nat], Nat.0, Nat.1)
    fs_insert(FiniteSet.empty[Nat], Nat.0).contains(Nat.1) =
        (Nat.1 = Nat.0 or FiniteSet.empty[Nat].contains(Nat.1))
    Nat.1 != Nat.0
    not FiniteSet.empty[Nat].contains(Nat.1)
    not fs_insert(FiniteSet.empty[Nat], Nat.0).contains(Nat.1)
    fs_card(s2) = fs_card(fs_insert(FiniteSet.empty[Nat], Nat.0)) + Nat.1
    fs_card(s2) = Nat.2
}

/// The three-element vertex set has three elements.
theorem s3_card {
    fs_card(s3) = Nat.3
} by {
    fs_card_insert_of_not_contains(FiniteSet.empty[Nat], Nat.0)
    not FiniteSet.empty[Nat].contains(Nat.0)
    fs_card(fs_insert(FiniteSet.empty[Nat], Nat.0)) = fs_card(FiniteSet.empty[Nat]) + Nat.1
    fs_card_empty[Nat]
    fs_card(FiniteSet.empty[Nat]) = Nat.0
    fs_card(fs_insert(FiniteSet.empty[Nat], Nat.0)) = Nat.1
    fs_card_insert_of_not_contains(fs_insert(FiniteSet.empty[Nat], Nat.0), Nat.1)
    fs_insert_contains_eq(FiniteSet.empty[Nat], Nat.0, Nat.1)
    fs_insert(FiniteSet.empty[Nat], Nat.0).contains(Nat.1) =
        (Nat.1 = Nat.0 or FiniteSet.empty[Nat].contains(Nat.1))
    Nat.1 != Nat.0
    not FiniteSet.empty[Nat].contains(Nat.1)
    not fs_insert(FiniteSet.empty[Nat], Nat.0).contains(Nat.1)
    fs_card(fs_insert(fs_insert(FiniteSet.empty[Nat], Nat.0), Nat.1)) =
        fs_card(fs_insert(FiniteSet.empty[Nat], Nat.0)) + Nat.1
    fs_card(fs_insert(fs_insert(FiniteSet.empty[Nat], Nat.0), Nat.1)) = Nat.2
    fs_card_insert_of_not_contains(fs_insert(fs_insert(FiniteSet.empty[Nat], Nat.0), Nat.1), Nat.2)
    fs_insert_not_contains_of_ne(fs_insert(FiniteSet.empty[Nat], Nat.0), Nat.1, Nat.2)
    Nat.2 != Nat.1
    fs_insert_not_contains_of_ne(FiniteSet.empty[Nat], Nat.0, Nat.2)
    Nat.2 != Nat.0
    not FiniteSet.empty[Nat].contains(Nat.2)
    not fs_insert(FiniteSet.empty[Nat], Nat.0).contains(Nat.2)
    Nat.2 != Nat.1 and not fs_insert(FiniteSet.empty[Nat], Nat.0).contains(Nat.2)
    not fs_insert(fs_insert(FiniteSet.empty[Nat], Nat.0), Nat.1).contains(Nat.2)
    fs_card(s3) = fs_card(fs_insert(fs_insert(FiniteSet.empty[Nat], Nat.0), Nat.1)) + Nat.1
    fs_card(s3) = Nat.3
}

/// The four-element vertex set has four elements.
theorem s4_card {
    fs_card(s4) = Nat.4
} by {
    fs_card_insert_of_not_contains(FiniteSet.empty[Nat], Nat.0)
    not FiniteSet.empty[Nat].contains(Nat.0)
    fs_card(fs_insert(FiniteSet.empty[Nat], Nat.0)) = fs_card(FiniteSet.empty[Nat]) + Nat.1
    fs_card_empty[Nat]
    fs_card(FiniteSet.empty[Nat]) = Nat.0
    fs_card(fs_insert(FiniteSet.empty[Nat], Nat.0)) = Nat.1
    fs_card_insert_of_not_contains(fs_insert(FiniteSet.empty[Nat], Nat.0), Nat.1)
    fs_insert_contains_eq(FiniteSet.empty[Nat], Nat.0, Nat.1)
    fs_insert(FiniteSet.empty[Nat], Nat.0).contains(Nat.1) =
        (Nat.1 = Nat.0 or FiniteSet.empty[Nat].contains(Nat.1))
    Nat.1 != Nat.0
    not FiniteSet.empty[Nat].contains(Nat.1)
    not fs_insert(FiniteSet.empty[Nat], Nat.0).contains(Nat.1)
    fs_card(fs_insert(fs_insert(FiniteSet.empty[Nat], Nat.0), Nat.1)) =
        fs_card(fs_insert(FiniteSet.empty[Nat], Nat.0)) + Nat.1
    fs_card(fs_insert(fs_insert(FiniteSet.empty[Nat], Nat.0), Nat.1)) = Nat.2
    fs_card_insert_of_not_contains(fs_insert(fs_insert(FiniteSet.empty[Nat], Nat.0), Nat.1), Nat.2)
    fs_insert_not_contains_of_ne(fs_insert(FiniteSet.empty[Nat], Nat.0), Nat.1, Nat.2)
    Nat.2 != Nat.1
    fs_insert_not_contains_of_ne(FiniteSet.empty[Nat], Nat.0, Nat.2)
    Nat.2 != Nat.0
    not FiniteSet.empty[Nat].contains(Nat.2)
    not fs_insert(FiniteSet.empty[Nat], Nat.0).contains(Nat.2)
    Nat.2 != Nat.1 and not fs_insert(FiniteSet.empty[Nat], Nat.0).contains(Nat.2)
    not fs_insert(fs_insert(FiniteSet.empty[Nat], Nat.0), Nat.1).contains(Nat.2)
    fs_card(fs_insert(fs_insert(fs_insert(FiniteSet.empty[Nat], Nat.0), Nat.1), Nat.2)) =
        fs_card(fs_insert(fs_insert(FiniteSet.empty[Nat], Nat.0), Nat.1)) + Nat.1
    fs_card(fs_insert(fs_insert(fs_insert(FiniteSet.empty[Nat], Nat.0), Nat.1), Nat.2)) = Nat.3
    fs_card_insert_of_not_contains(fs_insert(fs_insert(fs_insert(FiniteSet.empty[Nat], Nat.0), Nat.1), Nat.2), Nat.3)
    fs_insert_not_contains_of_ne(fs_insert(fs_insert(FiniteSet.empty[Nat], Nat.0), Nat.1), Nat.2, Nat.3)
    Nat.3 != Nat.2
    fs_insert_not_contains_of_ne(fs_insert(FiniteSet.empty[Nat], Nat.0), Nat.1, Nat.3)
    Nat.3 != Nat.1
    fs_insert_not_contains_of_ne(FiniteSet.empty[Nat], Nat.0, Nat.3)
    Nat.3 != Nat.0
    not FiniteSet.empty[Nat].contains(Nat.3)
    not fs_insert(FiniteSet.empty[Nat], Nat.0).contains(Nat.3)
    Nat.3 != Nat.1 and not fs_insert(FiniteSet.empty[Nat], Nat.0).contains(Nat.3)
    not fs_insert(fs_insert(FiniteSet.empty[Nat], Nat.0), Nat.1).contains(Nat.3)
    Nat.3 != Nat.2 and not fs_insert(fs_insert(FiniteSet.empty[Nat], Nat.0), Nat.1).contains(Nat.3)
    not fs_insert(fs_insert(fs_insert(FiniteSet.empty[Nat], Nat.0), Nat.1), Nat.2).contains(Nat.3)
    fs_card(s4) = fs_card(fs_insert(fs_insert(fs_insert(FiniteSet.empty[Nat], Nat.0), Nat.1), Nat.2)) + Nat.1
    fs_card(s4) = Nat.4
}

/// The complete graph on any vertex set is connected on that set.
///
/// Distinct vertices are joined by the edge between them; a vertex is joined to
/// itself by reflexive reachability. No hypothesis on the set is needed.
theorem complete_graph_connected_on_set[V](s: FiniteSet[V]) {
    simple_graph_connected_on(complete_graph[V], s)
} by {
    simple_graph_connected_on(complete_graph[V], s) = forall(x: V, y: V) {
        s.contains(x) and s.contains(y) implies
            simple_graph_reachable(induced_subgraph(complete_graph[V], s.underlying_set), x, y)
    }
    forall(x: V, y: V) {
        if s.contains(x) and s.contains(y) {
            if x = y {
                simple_graph_reachable_refl(induced_subgraph(complete_graph[V], s.underlying_set), x)
                simple_graph_reachable(induced_subgraph(complete_graph[V], s.underlying_set), x, y)
            } else {
                x != y
                complete_graph_adj_iff_ne[V](x, y)
                complete_graph[V].adj(x, y)
                s.underlying_set.contains(x)
                s.underlying_set.contains(y)
                induced_subgraph_adj_of_base_adj(complete_graph[V], s.underlying_set, x, y)
                induced_subgraph(complete_graph[V], s.underlying_set).adj(x, y)
                simple_graph_reachable_of_adj(induced_subgraph(complete_graph[V], s.underlying_set), x, y)
                simple_graph_reachable(induced_subgraph(complete_graph[V], s.underlying_set), x, y)
            }
        }
    }
    simple_graph_connected_on(complete_graph[V], s)
}

/// The complete graph on two vertices is a tree: a single edge.
theorem complete_graph_two_is_tree {
    is_tree(complete_graph[Nat], s2)
} by {
    is_tree(complete_graph[Nat], s2) =
        (simple_graph_connected_on(complete_graph[Nat], s2) and
            is_acyclic(complete_graph[Nat], s2))
    complete_graph_connected_on_set(s2)
    simple_graph_connected_on(complete_graph[Nat], s2)
    is_acyclic_of_card_le_two(complete_graph[Nat], s2)
    fs_card(s2) <= Nat.2
    is_acyclic(complete_graph[Nat], s2)
    simple_graph_connected_on(complete_graph[Nat], s2) and
        is_acyclic(complete_graph[Nat], s2)
}

/// The tree on two vertices has one edge.
theorem edge_count_tree {
    edge_count(complete_graph[Nat], s2) = Nat.1
} by {
    s2_card
    fs_card(s2) = Nat.2
    edge_count_complete_of_card(s2, Nat.2)
    edge_count(complete_graph[Nat], s2) = ((Nat.2 - Nat.1) * Nat.2).div(Nat.2)
    suc_sub_one(Nat.1)
    Nat.2 - Nat.1 = Nat.1
    div_mul(Nat.1, Nat.2)
    Nat.2 != Nat.0
    (Nat.1 * Nat.2).div(Nat.2) = Nat.1
    ((Nat.2 - Nat.1) * Nat.2).div(Nat.2) = Nat.1
    edge_count(complete_graph[Nat], s2) = Nat.1
}

/// The cycle `C3` (a triangle) has three edges.
theorem edge_count_cycle3 {
    edge_count(complete_graph[Nat], s3) = Nat.3
} by {
    s3_card
    fs_card(s3) = Nat.3
    edge_count_complete_of_card(s3, Nat.3)
    edge_count(complete_graph[Nat], s3) = ((Nat.3 - Nat.1) * Nat.3).div(Nat.2)
    suc_sub_one(Nat.2)
    Nat.3 - Nat.1 = Nat.2
    mul_comm(Nat.3, Nat.2)
    Nat.2 * Nat.3 = Nat.3 * Nat.2
    div_mul(Nat.3, Nat.2)
    (Nat.3 * Nat.2).div(Nat.2) = Nat.3
    (Nat.2 * Nat.3).div(Nat.2) = Nat.3
    ((Nat.3 - Nat.1) * Nat.3).div(Nat.2) = Nat.3
    edge_count(complete_graph[Nat], s3) = Nat.3
}

/// The complete graph `K4` has six edges.
theorem edge_count_k4 {
    edge_count(complete_graph[Nat], s4) = Nat.6
} by {
    s4_card
    fs_card(s4) = Nat.4
    edge_count_complete_of_card(s4, Nat.4)
    edge_count(complete_graph[Nat], s4) = ((Nat.4 - Nat.1) * Nat.4).div(Nat.2)
    suc_sub_one(Nat.3)
    Nat.4 - Nat.1 = Nat.3
    mul_comm(Nat.6, Nat.2)
    Nat.2 * Nat.6 = Nat.6 * Nat.2
    div_mul(Nat.6, Nat.2)
    (Nat.6 * Nat.2).div(Nat.2) = Nat.6
    (Nat.2 * Nat.6).div(Nat.2) = Nat.6
    (Nat.3 * Nat.4).div(Nat.2) = (Nat.2 * Nat.6).div(Nat.2)
    ((Nat.4 - Nat.1) * Nat.4).div(Nat.2) = Nat.6
    edge_count(complete_graph[Nat], s4) = Nat.6
}

/// Euler's formula for a tree: with `V` vertices, `E` edges and one face,
/// `V + 1 = E + 2`.
///
/// The concrete witness is the tree on two vertices (`K2`): `V = 2`, `E = 1`,
/// `F = 1`, so `2 + 1 = 1 + 2`.
theorem euler_formula_tree {
    is_tree(complete_graph[Nat], s2) implies
        fs_card(s2) + Nat.1 = edge_count(complete_graph[Nat], s2) + Nat.2
} by {
    if is_tree(complete_graph[Nat], s2) {
        s2_card
        fs_card(s2) = Nat.2
        edge_count_tree
        edge_count(complete_graph[Nat], s2) = Nat.1
        Nat.2 + Nat.1 = Nat.1 + Nat.2
        fs_card(s2) + Nat.1 = edge_count(complete_graph[Nat], s2) + Nat.2
    }
}

/// Euler's formula for the cycle `C3` (a triangle): `V = 3`, `E = 3`, `F = 2`,
/// so `3 + 2 = 3 + 2`.
theorem euler_formula_cycle3 {
    fs_card(s3) + Nat.2 = edge_count(complete_graph[Nat], s3) + Nat.2
} by {
    s3_card
    fs_card(s3) = Nat.3
    edge_count_cycle3
    edge_count(complete_graph[Nat], s3) = Nat.3
    Nat.3 + Nat.2 = Nat.3 + Nat.2
    fs_card(s3) + Nat.2 = edge_count(complete_graph[Nat], s3) + Nat.2
}

/// Euler's formula for `K4`: `V = 4`, `E = 6`, `F = 4`, so `4 + 4 = 6 + 2`.
theorem euler_formula_k4 {
    fs_card(s4) + Nat.4 = edge_count(complete_graph[Nat], s4) + Nat.2
} by {
    s4_card
    fs_card(s4) = Nat.4
    edge_count_k4
    edge_count(complete_graph[Nat], s4) = Nat.6
    Nat.4 + Nat.4 = Nat.6 + Nat.2
    fs_card(s4) + Nat.4 = edge_count(complete_graph[Nat], s4) + Nat.2
}

// ------------------------------------------------------------
// Corollary
// ------------------------------------------------------------

/// Corollary of Euler's formula: a simple planar graph on at least three vertices
/// satisfies `E <= 3V - 6`.
///
/// The statement is the standard edge bound for simple planar graphs. It is vacuous
/// under the placeholder `is_planar`; the classical proof, which becomes an
/// obligation when a genuine embedding definition replaces the placeholder, doubles
/// the face-edge incidences: in a simple planar graph every face has length at
/// least three and every edge borders at most two faces, so `2E >= 3F`, and Euler's
/// formula `F = E - V + 2` then gives `2E >= 3(E - V + 2)`, i.e. `E <= 3V - 6`.
theorem planar_edge_bound[V](g: SimpleGraph[V], s: FiniteSet[V]) {
    is_planar(g) and Nat.3 <= fs_card(s) implies
        edge_count(g, s) <= Nat.3 * fs_card(s) - Nat.6
} by {
    if is_planar(g) and Nat.3 <= fs_card(s) {
        is_planar(g)
        is_planar(g) = false
        false
    }
    (is_planar(g) and Nat.3 <= fs_card(s)) = false
    false_implies(edge_count(g, s) <= Nat.3 * fs_card(s) - Nat.6)
    (false implies edge_count(g, s) <= Nat.3 * fs_card(s) - Nat.6) = true
    ((is_planar(g) and Nat.3 <= fs_card(s)) implies
        edge_count(g, s) <= Nat.3 * fs_card(s) - Nat.6)
}
// Future work: the four-cycle `C4` (a square) and the general cycle `C_n`, whose
// edge counts need a per-vertex case analysis over the labels; the triangle `C3`
// above already instantiates the cycle base case `V = E`, `F = 2`. The larger
// next step is a genuine planar embedding, with a real `is_planar` predicate and
// face counts as regions of the embedding.
