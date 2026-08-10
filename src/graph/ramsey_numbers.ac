// ============================================================================
// Ramsey numbers: the recurrence and its first consequences.
//
// This file records the general Ramsey-number framework and the classical
// upper bounds that follow from the recurrence
//
//     R(k, l) <= R(k - 1, l) + R(k, l - 1),
//
// together with the Erdős–Szekeres theorem on monotone subsequences (whose
// general form lives in combinatorics/erdos_szekeres.ac).
//
// A Ramsey number bound R(k, l) <= n is stated as `ramsey_number_upper(k, l,
// n)`: every red/blue 2-coloring of the edges of the complete graph on n
// vertices has a red clique of size k or a blue clique of size l.  The
// classical proofs of R(3, 5) <= 14 and R(4, 4) <= 18 reduce to arithmetic
// once the recurrence is granted: R(3, 5) <= R(2, 5) + R(3, 4) = 5 + 9 = 14
// and R(4, 4) <= R(3, 4) + R(4, 3) = 9 + 9 = 18.  Those arithmetic
// consequences are verified here; the recurrence itself is stated as
// `ramsey_recurrence`, whose full proof (pigeonhole on the neighbours of a
// vertex, then the two sub-bounds applied to the two monochromatic
// neighbourhoods) is recorded commented out below.
// ============================================================================

from nat import Nat, not_lt_zero, lt_suc_right, zero_or_suc, trichotomy, lt_cancel_suc,
    lt_and_lte, lte_and_lt, lt_imp_lte_suc, lte_cancel_suc, lt_suc, suc_sub_one, add_sub,
    sub_zero, lte_add_left, mul_comm, lte_imp_not_lt, only_zero_lte_zero, lte_trans,
    lt_add_suc, add_zero_left, add_suc_left, lt_or_lte, lt_not_ref, mul_suc_right
from finite_set import FiniteSet
from list import List, singleton_unique, singleton_contains_imp_eq, map, length_range,
    range_is_unique, lt_of_range_contains, range_contains_of_lt,
    unique_is_smallest_containing_list, unique_implies_no_duplicate,
    list_contains_implies_count_geq_one, map_contains
from data.list.list_cons_membership import cons_contains_cases, cons_contains_head,
    cons_contains_of_tail_contains, nil_not_contains
from data.list.list_pigeonhole import pigeonhole_unique_map_in
from data.basic.logic import exists_intro
from data.nat.nat_range_set import range_set, range_set_contains
from graph.simple_graph import complete_graph, complete_graph_adj_iff_ne
from graph.ramsey import is_edge_2_coloring, edge_2_coloring_comm, has_red_triangle,
    has_blue_triangle, has_mono_triangle, six_vertices, ramsey_r33_upper
from graph.ramsey_numbers_esz import value_at, get_idx_cons_zero, get_idx_cons_suc,
    option_some_eq_iff, value_at_cons_zero, value_at_cons_suc, esz_get_idx_prop,
    esz_get_idx_prop_step, esz_value_at_prop, esz_value_at_prop_step
from pair import Pair, pair_eta, pair_ext, pair_new_first, pair_new_second
from order import lt_trans, lte_or_lte_swap

numerals Nat

/// A red clique of size k: k distinct elements of `s`, pairwise joined by red edges.
///
/// The clique is witnessed by a list of k distinct members of `s`; the pairwise
/// condition requires of every two members that they are the same element or
/// joined by a red edge (a single orientation suffices, exactly as in
/// `has_red_triangle`).  With `xs.is_unique` this says precisely that any two
/// distinct members are red-joined.
/// All members of the list lie in `s`.
define clique_members[V](xs: List[V], s: FiniteSet[V]) -> Bool {
    forall(x: V) {
        xs.contains(x) implies s.contains(x)
    }
}

/// Any two members of the list are the same element or red-joined.
define clique_pairwise_red[V](color: (V, V) -> Bool, xs: List[V]) -> Bool {
    forall(x: V, y: V) {
        xs.contains(x) and xs.contains(y) implies (x = y or color(x, y))
    }
}

/// Any two members of the list are the same element or blue-joined.
define clique_pairwise_blue[V](color: (V, V) -> Bool, xs: List[V]) -> Bool {
    forall(x: V, y: V) {
        xs.contains(x) and xs.contains(y) implies (x = y or not color(x, y))
    }
}

/// A red clique of size k: k distinct elements of `s`, pairwise joined by red edges.
///
/// The clique is witnessed by a list of k distinct members of `s` satisfying
/// `clique_members` (all members lie in `s`) and `clique_pairwise_red` (every
/// two members are the same element or red-joined; with `xs.is_unique` this
/// says precisely that any two distinct members are red-joined).
define has_red_clique[V](color: (V, V) -> Bool, s: FiniteSet[V], k: Nat) -> Bool {
    exists(xs: List[V]) {
        xs.length = k and xs.is_unique and clique_members(xs, s) and clique_pairwise_red(color, xs)
    }
}

/// A blue clique of size k: k distinct elements of `s`, pairwise joined by blue edges.
/// A blue clique of size k: k distinct elements of `s`, pairwise joined by blue edges.
define has_blue_clique[V](color: (V, V) -> Bool, s: FiniteSet[V], k: Nat) -> Bool {
    exists(xs: List[V]) {
        xs.length = k and xs.is_unique and clique_members(xs, s) and clique_pairwise_blue(color, xs)
    }
}

/// The Ramsey number bound R(k, l) <= n: every red/blue 2-coloring of the
/// complete graph on `n` vertices has a red clique of size `k` or a blue
/// clique of size `l`.
define ramsey_number_upper(k: Nat, l: Nat, n: Nat) -> Bool {
    forall(color: (Nat, Nat) -> Bool) {
        is_edge_2_coloring(complete_graph[Nat], color) implies
        (has_red_clique(color, range_set(n), k) or has_blue_clique(color, range_set(n), l))
    }
}

/// The Ramsey recurrence R(k, l) <= R(k - 1, l) + R(k, l - 1) in predicate
/// form: if R(k - 1, l) <= a and R(k, l - 1) <= b, then R(k, l) <= a + b.
///
/// The classical proof fixes a vertex v of K_{a + b}; among its a + b - 1
/// neighbours either a share the colour of the edge from v (in which case the
/// bound R(k - 1, l) <= a applied to those neighbours gives a red K_{k - 1},
/// which closes with v, or a blue K_l) or b are opposite-coloured (and the
/// bound R(k, l - 1) <= b gives a red K_k or a blue K_{l - 1}, which closes
/// with v).  The two sub-bounds are stated over the fixed vertex sets
/// `range_set(a)` and `range_set(b)` in `ramsey_number_upper`, so applying
/// them to the neighbourhoods of v needs the transport of the statement along
/// an order-preserving bijection; that step is not yet verified, and the
/// theorem is recorded as a comment at the end of this file.
define ramsey_recurrence(k: Nat, l: Nat, a: Nat, b: Nat) -> Bool {
    ramsey_number_upper(k - Nat.1, l, a) and ramsey_number_upper(k, l - Nat.1, b)
    implies ramsey_number_upper(k, l, a + b)
}

// ============================================================================
// Arithmetic corollaries of the recurrence.
// ============================================================================

/// R(3, 5) <= R(2, 5) + R(3, 4) = 5 + 9 = 14: the arithmetic content of the
/// bound R(3, 5) <= 14.
theorem ramsey_r35_bound_arith {
    Nat.14 = Nat.5 + Nat.9
} by {
    add_suc_left(Nat.4, Nat.9)
    Nat.4.suc + Nat.9 = (Nat.4 + Nat.9).suc
    Nat.4.suc = Nat.5
    Nat.5 + Nat.9 = (Nat.4 + Nat.9).suc
    add_suc_left(Nat.3, Nat.9)
    Nat.3.suc + Nat.9 = (Nat.3 + Nat.9).suc
    Nat.3.suc = Nat.4
    Nat.4 + Nat.9 = (Nat.3 + Nat.9).suc
    Nat.5 + Nat.9 = (Nat.3 + Nat.9).suc.suc
    add_suc_left(Nat.2, Nat.9)
    Nat.2.suc + Nat.9 = (Nat.2 + Nat.9).suc
    Nat.2.suc = Nat.3
    Nat.3 + Nat.9 = (Nat.2 + Nat.9).suc
    Nat.5 + Nat.9 = (Nat.2 + Nat.9).suc.suc.suc
    add_suc_left(Nat.1, Nat.9)
    Nat.1.suc + Nat.9 = (Nat.1 + Nat.9).suc
    Nat.1.suc = Nat.2
    Nat.2 + Nat.9 = (Nat.1 + Nat.9).suc
    Nat.5 + Nat.9 = (Nat.1 + Nat.9).suc.suc.suc.suc
    add_suc_left(Nat.0, Nat.9)
    Nat.0.suc + Nat.9 = (Nat.0 + Nat.9).suc
    Nat.0.suc = Nat.1
    Nat.1 + Nat.9 = (Nat.0 + Nat.9).suc
    Nat.5 + Nat.9 = (Nat.0 + Nat.9).suc.suc.suc.suc.suc
    add_zero_left(Nat.9)
    Nat.0 + Nat.9 = Nat.9
    Nat.5 + Nat.9 = Nat.9.suc.suc.suc.suc.suc
    Nat.9.suc = Nat.10
    Nat.10.suc = Nat.11
    Nat.11.suc = Nat.12
    Nat.12.suc = Nat.13
    Nat.13.suc = Nat.14
    Nat.9.suc.suc.suc.suc.suc = Nat.14
    Nat.5 + Nat.9 = Nat.14
    Nat.14 = Nat.5 + Nat.9
}

/// R(4, 4) <= R(3, 4) + R(4, 3) = 9 + 9 = 18: the arithmetic content of the
/// bound R(4, 4) <= 18.  The two summands are equal because R(4, 3) = R(3, 4)
/// by symmetry of the two colours.
theorem ramsey_r44_bound_arith {
    Nat.18 = Nat.9 + Nat.9
} by {
    add_suc_left(Nat.8, Nat.9)
    Nat.8.suc + Nat.9 = (Nat.8 + Nat.9).suc
    Nat.8.suc = Nat.9
    Nat.9 + Nat.9 = (Nat.8 + Nat.9).suc
    add_suc_left(Nat.7, Nat.9)
    Nat.7.suc + Nat.9 = (Nat.7 + Nat.9).suc
    Nat.7.suc = Nat.8
    Nat.8 + Nat.9 = (Nat.7 + Nat.9).suc
    Nat.9 + Nat.9 = (Nat.7 + Nat.9).suc.suc
    add_suc_left(Nat.6, Nat.9)
    Nat.6.suc + Nat.9 = (Nat.6 + Nat.9).suc
    Nat.6.suc = Nat.7
    Nat.7 + Nat.9 = (Nat.6 + Nat.9).suc
    Nat.9 + Nat.9 = (Nat.6 + Nat.9).suc.suc.suc
    add_suc_left(Nat.5, Nat.9)
    Nat.5.suc + Nat.9 = (Nat.5 + Nat.9).suc
    Nat.5.suc = Nat.6
    Nat.6 + Nat.9 = (Nat.5 + Nat.9).suc
    Nat.9 + Nat.9 = (Nat.5 + Nat.9).suc.suc.suc.suc
    add_suc_left(Nat.4, Nat.9)
    Nat.4.suc + Nat.9 = (Nat.4 + Nat.9).suc
    Nat.4.suc = Nat.5
    Nat.5 + Nat.9 = (Nat.4 + Nat.9).suc
    Nat.9 + Nat.9 = (Nat.4 + Nat.9).suc.suc.suc.suc.suc
    add_suc_left(Nat.3, Nat.9)
    Nat.3.suc + Nat.9 = (Nat.3 + Nat.9).suc
    Nat.3.suc = Nat.4
    Nat.4 + Nat.9 = (Nat.3 + Nat.9).suc
    Nat.9 + Nat.9 = (Nat.3 + Nat.9).suc.suc.suc.suc.suc.suc
    add_suc_left(Nat.2, Nat.9)
    Nat.2.suc + Nat.9 = (Nat.2 + Nat.9).suc
    Nat.2.suc = Nat.3
    Nat.3 + Nat.9 = (Nat.2 + Nat.9).suc
    Nat.9 + Nat.9 = (Nat.2 + Nat.9).suc.suc.suc.suc.suc.suc.suc
    add_suc_left(Nat.1, Nat.9)
    Nat.1.suc + Nat.9 = (Nat.1 + Nat.9).suc
    Nat.1.suc = Nat.2
    Nat.2 + Nat.9 = (Nat.1 + Nat.9).suc
    Nat.9 + Nat.9 = (Nat.1 + Nat.9).suc.suc.suc.suc.suc.suc.suc.suc
    add_suc_left(Nat.0, Nat.9)
    Nat.0.suc + Nat.9 = (Nat.0 + Nat.9).suc
    Nat.0.suc = Nat.1
    Nat.1 + Nat.9 = (Nat.0 + Nat.9).suc
    Nat.9 + Nat.9 = (Nat.0 + Nat.9).suc.suc.suc.suc.suc.suc.suc.suc.suc
    add_zero_left(Nat.9)
    Nat.0 + Nat.9 = Nat.9
    Nat.9 + Nat.9 = Nat.9.suc.suc.suc.suc.suc.suc.suc.suc.suc
    Nat.9.suc = Nat.10
    Nat.10.suc = Nat.11
    Nat.11.suc = Nat.12
    Nat.12.suc = Nat.13
    Nat.13.suc = Nat.14
    Nat.14.suc = Nat.15
    Nat.15.suc = Nat.16
    Nat.16.suc = Nat.17
    Nat.17.suc = Nat.18
    Nat.9.suc.suc.suc.suc.suc.suc.suc.suc.suc = Nat.18
    Nat.9 + Nat.9 = Nat.18
    Nat.18 = Nat.9 + Nat.9
}

/// The recurrence at (k, l) = (3, 5) with a = 5 and b = 9: granted the two
/// sub-bounds R(2, 5) <= 5 and R(3, 4) <= 9, the recurrence reduces R(3, 5) to
/// the sum 5 + 9 = 14.  The proof unpacks the recurrence at these numbers and
/// closes the arithmetic (3 - 1 = 2, 5 - 1 = 4, 5 + 9 = 14).
theorem ramsey_r35_from_recurrence {
    ramsey_number_upper(Nat.2, Nat.5, Nat.5) and
    ramsey_number_upper(Nat.3, Nat.4, Nat.9) and
    ramsey_recurrence(Nat.3, Nat.5, Nat.5, Nat.9)
    implies ramsey_number_upper(Nat.3, Nat.5, Nat.14)
} by {
    if ramsey_number_upper(Nat.2, Nat.5, Nat.5) and
        ramsey_number_upper(Nat.3, Nat.4, Nat.9) and
        ramsey_recurrence(Nat.3, Nat.5, Nat.5, Nat.9) {
        ramsey_recurrence(Nat.3, Nat.5, Nat.5, Nat.9) =
            (ramsey_number_upper(Nat.3 - Nat.1, Nat.5, Nat.5)
                and ramsey_number_upper(Nat.3, Nat.5 - Nat.1, Nat.9)
                implies ramsey_number_upper(Nat.3, Nat.5, Nat.5 + Nat.9))
        Nat.3 - Nat.1 = Nat.2
        Nat.5 - Nat.1 = Nat.4
        ramsey_number_upper(Nat.3 - Nat.1, Nat.5, Nat.5)
        ramsey_number_upper(Nat.3, Nat.5 - Nat.1, Nat.9)
        ramsey_number_upper(Nat.3 - Nat.1, Nat.5, Nat.5) and
            ramsey_number_upper(Nat.3, Nat.5 - Nat.1, Nat.9)
        ramsey_number_upper(Nat.3, Nat.5, Nat.5 + Nat.9)
        ramsey_r35_bound_arith
        Nat.14 = Nat.5 + Nat.9
        Nat.5 + Nat.9 = Nat.14
        ramsey_number_upper(Nat.3, Nat.5, Nat.5 + Nat.9) =
            ramsey_number_upper(Nat.3, Nat.5, Nat.14)
        ramsey_number_upper(Nat.3, Nat.5, Nat.14)
    }
}

/// The recurrence at (k, l) = (4, 4) with a = b = 9: granted the two sub-bounds
/// R(3, 4) <= 9 and R(4, 3) <= 9, the recurrence reduces R(4, 4) to the sum
/// 9 + 9 = 18.  The proof unpacks the recurrence at these numbers and closes
/// the arithmetic (4 - 1 = 3, 9 + 9 = 18).
theorem ramsey_r44_from_recurrence {
    ramsey_number_upper(Nat.3, Nat.4, Nat.9) and
    ramsey_number_upper(Nat.4, Nat.3, Nat.9) and
    ramsey_recurrence(Nat.4, Nat.4, Nat.9, Nat.9)
    implies ramsey_number_upper(Nat.4, Nat.4, Nat.18)
} by {
    if ramsey_number_upper(Nat.3, Nat.4, Nat.9) and
        ramsey_number_upper(Nat.4, Nat.3, Nat.9) and
        ramsey_recurrence(Nat.4, Nat.4, Nat.9, Nat.9) {
        ramsey_recurrence(Nat.4, Nat.4, Nat.9, Nat.9) =
            (ramsey_number_upper(Nat.4 - Nat.1, Nat.4, Nat.9)
                and ramsey_number_upper(Nat.4, Nat.4 - Nat.1, Nat.9)
                implies ramsey_number_upper(Nat.4, Nat.4, Nat.9 + Nat.9))
        Nat.4 - Nat.1 = Nat.3
        ramsey_number_upper(Nat.4 - Nat.1, Nat.4, Nat.9)
        ramsey_number_upper(Nat.4, Nat.4 - Nat.1, Nat.9)
        ramsey_number_upper(Nat.4 - Nat.1, Nat.4, Nat.9) and
            ramsey_number_upper(Nat.4, Nat.4 - Nat.1, Nat.9)
        ramsey_number_upper(Nat.4, Nat.4, Nat.9 + Nat.9)
        ramsey_r44_bound_arith
        Nat.18 = Nat.9 + Nat.9
        Nat.9 + Nat.9 = Nat.18
        ramsey_number_upper(Nat.4, Nat.4, Nat.9 + Nat.9) =
            ramsey_number_upper(Nat.4, Nat.4, Nat.18)
        ramsey_number_upper(Nat.4, Nat.4, Nat.18)
    }
}

// ============================================================================
// The special case R(3, 3) <= 6 in the new framework.
//
// The library's `ramsey_r33_upper` (graph/ramsey.ac) proves R(3, 3) <= 6 in
// the triangle form: every 2-coloring of the complete graph on six vertices
// has a monochromatic triangle.  The bridge theorems below connect that form
// to the clique form `ramsey_number_upper(3, 3, 6)` used in this file: a
// monochromatic triangle is a 3-element red (resp. blue) clique, witnessed by
// the list of its three vertices.
// ============================================================================

/// The length of the three-element witness list is three.
theorem three_list_length[T](x: T, y: T, z: T) {
    List.cons(x, List.cons(y, List.cons(z, List.nil[T]))).length = Nat.3
} by {
    List.nil[T].length = Nat.0
    List.cons(z, List.nil[T]).length = List.nil[T].length.suc
    List.cons(z, List.nil[T]).length = Nat.0.suc
    Nat.0.suc = Nat.1
    List.cons(z, List.nil[T]).length = Nat.1
    List.cons(y, List.cons(z, List.nil[T])).length = List.cons(z, List.nil[T]).length.suc
    List.cons(y, List.cons(z, List.nil[T])).length = Nat.1.suc
    Nat.1.suc = Nat.2
    List.cons(y, List.cons(z, List.nil[T])).length = Nat.2
    List.cons(x, List.cons(y, List.cons(z, List.nil[T]))).length =
        List.cons(y, List.cons(z, List.nil[T])).length.suc
    List.cons(x, List.cons(y, List.cons(z, List.nil[T]))).length = Nat.2.suc
    Nat.2.suc = Nat.3
    List.cons(x, List.cons(y, List.cons(z, List.nil[T]))).length = Nat.3
}

/// A member of the three-element witness list is one of its three vertices.
theorem three_list_contains_cases[T](x: T, y: T, z: T, a: T) {
    List.cons(x, List.cons(y, List.cons(z, List.nil[T]))).contains(a)
        implies a = x or a = y or a = z
} by {
    if List.cons(x, List.cons(y, List.cons(z, List.nil[T]))).contains(a) {
        cons_contains_cases(x, List.cons(y, List.cons(z, List.nil[T])), a)
        a = x or List.cons(y, List.cons(z, List.nil[T])).contains(a)
        if a = x {
            a = x or a = y or a = z
        } else {
            List.cons(y, List.cons(z, List.nil[T])).contains(a)
            cons_contains_cases(y, List.cons(z, List.nil[T]), a)
            a = y or List.cons(z, List.nil[T]).contains(a)
            if a = y {
                a = x or a = y or a = z
            } else {
                List.cons(z, List.nil[T]).contains(a)
                cons_contains_cases(z, List.nil[T], a)
                a = z or List.nil[T].contains(a)
                nil_not_contains(a)
                not List.nil[T].contains(a)
                if a = z {
                    a = x or a = y or a = z
                } else {
                    List.nil[T].contains(a)
                    false
                }
                a = x or a = y or a = z
            }
            a = x or a = y or a = z
        }
        a = x or a = y or a = z
    }
}

/// Three pairwise distinct elements form a unique three-element list.
///
/// `is_unique` is `self.unique = self`, so the proof computes `unique` on the
/// three-element list: each cons step keeps its head because the tail does not
/// contain it.
theorem three_list_unique[T](x: T, y: T, z: T) {
    x != y and x != z and y != z implies
    List.cons(x, List.cons(y, List.cons(z, List.nil[T]))).is_unique
} by {
    if x != y and x != z and y != z {
        List.nil[T].unique = List.nil[T]
        List.nil[T].contains(z) = false
        List.cons(z, List.nil[T]).unique = List.cons(z, List.nil[T].unique)
        List.cons(z, List.nil[T]).unique = List.cons(z, List.nil[T])
        if List.singleton(z).contains(y) {
            singleton_contains_imp_eq(z, y)
            y = z
            false
        }
        not List.singleton(z).contains(y)
        List.cons(y, List.singleton(z)).unique = List.cons(y, List.singleton(z).unique)
        List.cons(y, List.singleton(z)).unique = List.cons(y, List.cons(z, List.nil[T]))
        if List.cons(y, List.singleton(z)).contains(x) {
            cons_contains_cases(y, List.singleton(z), x)
            y = x or List.singleton(z).contains(x)
            if y = x {
                false
            } else {
                List.singleton(z).contains(x)
                singleton_contains_imp_eq(z, x)
                x = z
                false
            }
            false
        }
        not List.cons(y, List.singleton(z)).contains(x)
        List.cons(x, List.cons(y, List.singleton(z))).unique =
            List.cons(x, List.cons(y, List.singleton(z)).unique)
        List.cons(x, List.cons(y, List.singleton(z))).unique =
            List.cons(x, List.cons(y, List.cons(z, List.nil[T])))
        List.cons(x, List.cons(y, List.cons(z, List.nil[T]))).unique =
            List.cons(x, List.cons(y, List.cons(z, List.nil[T])))
        List.cons(x, List.cons(y, List.cons(z, List.nil[T]))).is_unique
    }
}

/// Three pairwise red edges among `x`, `y`, `z`: every two of the three
/// vertices are the same vertex or joined by a red edge (using symmetry of the
/// coloring for the reversed pairs).
theorem red_pairwise_three[V](color: (V, V) -> Bool, x: V, y: V, z: V, a: V, b: V) {
    is_edge_2_coloring(complete_graph[V], color) and
    color(x, y) and color(x, z) and color(y, z) and
    (a = x or a = y or a = z) and (b = x or b = y or b = z)
    implies (a = b or color(a, b))
} by {
    if is_edge_2_coloring(complete_graph[V], color) and
        color(x, y) and color(x, z) and color(y, z) and
        (a = x or a = y or a = z) and (b = x or b = y or b = z) {
        edge_2_coloring_comm(complete_graph[V], color, x, y)
        edge_2_coloring_comm(complete_graph[V], color, x, z)
        edge_2_coloring_comm(complete_graph[V], color, y, z)
        complete_graph_adj_iff_ne[V](x, y)
        complete_graph[V].adj(x, y) = (x != y)
        complete_graph_adj_iff_ne[V](x, z)
        complete_graph[V].adj(x, z) = (x != z)
        complete_graph_adj_iff_ne[V](y, z)
        complete_graph[V].adj(y, z) = (y != z)
        if a = x {
            if b = x {
                a = b
                a = b or color(a, b)
            } else {
                if b = y {
                    color(x, y)
                    color(a, b)
                    a = b or color(a, b)
                } else {
                    b = z
                    color(x, z)
                    color(a, b)
                    a = b or color(a, b)
                }
                a = b or color(a, b)
            }
            a = b or color(a, b)
        } else {
            if a = y {
                if b = x {
                    x != y
                    complete_graph[V].adj(x, y)
                    edge_2_coloring_comm(complete_graph[V], color, x, y)
                    color(x, y) = color(y, x)
                    color(y, x)
                    color(a, b)
                    a = b or color(a, b)
                } else {
                    if b = y {
                        a = b
                        a = b or color(a, b)
                    } else {
                        b = z
                        color(y, z)
                        color(a, b)
                        a = b or color(a, b)
                    }
                    a = b or color(a, b)
                }
                a = b or color(a, b)
            } else {
                a = z
                if b = x {
                    x != z
                    complete_graph[V].adj(x, z)
                    edge_2_coloring_comm(complete_graph[V], color, x, z)
                    color(x, z) = color(z, x)
                    color(z, x)
                    color(a, b)
                    a = b or color(a, b)
                } else {
                    if b = y {
                        y != z
                        complete_graph[V].adj(y, z)
                        edge_2_coloring_comm(complete_graph[V], color, y, z)
                        color(y, z) = color(z, y)
                        color(z, y)
                        color(a, b)
                        a = b or color(a, b)
                    } else {
                        b = z
                        a = b
                        a = b or color(a, b)
                    }
                    a = b or color(a, b)
                }
                a = b or color(a, b)
            }
            a = b or color(a, b)
        }
        a = b or color(a, b)
    }
}

/// Three pairwise blue edges among `x`, `y`, `z`: every two of the three
/// vertices are the same vertex or joined by a blue edge (using symmetry of
/// the coloring for the reversed pairs).
theorem blue_pairwise_three[V](color: (V, V) -> Bool, x: V, y: V, z: V, a: V, b: V) {
    is_edge_2_coloring(complete_graph[V], color) and
    not color(x, y) and not color(x, z) and not color(y, z) and
    (a = x or a = y or a = z) and (b = x or b = y or b = z)
    implies (a = b or not color(a, b))
} by {
    if is_edge_2_coloring(complete_graph[V], color) and
        not color(x, y) and not color(x, z) and not color(y, z) and
        (a = x or a = y or a = z) and (b = x or b = y or b = z) {
        edge_2_coloring_comm(complete_graph[V], color, x, y)
        edge_2_coloring_comm(complete_graph[V], color, x, z)
        edge_2_coloring_comm(complete_graph[V], color, y, z)
        complete_graph_adj_iff_ne[V](x, y)
        complete_graph[V].adj(x, y) = (x != y)
        complete_graph_adj_iff_ne[V](x, z)
        complete_graph[V].adj(x, z) = (x != z)
        complete_graph_adj_iff_ne[V](y, z)
        complete_graph[V].adj(y, z) = (y != z)
        if a = x {
            if b = x {
                a = b
                a = b or not color(a, b)
            } else {
                if b = y {
                    not color(x, y)
                    not color(a, b)
                    a = b or not color(a, b)
                } else {
                    b = z
                    not color(x, z)
                    not color(a, b)
                    a = b or not color(a, b)
                }
                a = b or not color(a, b)
            }
            a = b or not color(a, b)
        } else {
            if a = y {
                if b = x {
                    x != y
                    complete_graph[V].adj(x, y)
                    edge_2_coloring_comm(complete_graph[V], color, x, y)
                    color(x, y) = color(y, x)
                    not color(y, x)
                    not color(a, b)
                    a = b or not color(a, b)
                } else {
                    if b = y {
                        a = b
                        a = b or not color(a, b)
                    } else {
                        b = z
                        not color(y, z)
                        not color(a, b)
                        a = b or not color(a, b)
                    }
                    a = b or not color(a, b)
                }
                a = b or not color(a, b)
            } else {
                a = z
                if b = x {
                    x != z
                    complete_graph[V].adj(x, z)
                    edge_2_coloring_comm(complete_graph[V], color, x, z)
                    color(x, z) = color(z, x)
                    not color(z, x)
                    not color(a, b)
                    a = b or not color(a, b)
                } else {
                    if b = y {
                        y != z
                        complete_graph[V].adj(y, z)
                        edge_2_coloring_comm(complete_graph[V], color, y, z)
                        color(y, z) = color(z, y)
                        not color(z, y)
                        not color(a, b)
                        a = b or not color(a, b)
                    } else {
                        b = z
                        a = b
                        a = b or not color(a, b)
                    }
                    a = b or not color(a, b)
                }
                a = b or not color(a, b)
            }
            a = b or not color(a, b)
        }
        a = b or not color(a, b)
    }
}

/// Two members of the red triangle's witness list are the same vertex or
/// red-joined, given that both lie in the list.
theorem red_pairwise_three_list[V](color: (V, V) -> Bool, x: V, y: V, z: V, u: V, v: V) {
    is_edge_2_coloring(complete_graph[V], color) and
    color(x, y) and color(x, z) and color(y, z) and
    List.cons(x, List.cons(y, List.cons(z, List.nil[V]))).contains(u) and
    List.cons(x, List.cons(y, List.cons(z, List.nil[V]))).contains(v)
    implies (u = v or color(u, v))
} by {
    if is_edge_2_coloring(complete_graph[V], color) and
        color(x, y) and color(x, z) and color(y, z) and
        List.cons(x, List.cons(y, List.cons(z, List.nil[V]))).contains(u) and
        List.cons(x, List.cons(y, List.cons(z, List.nil[V]))).contains(v) {
        edge_2_coloring_comm(complete_graph[V], color, x, y)
        edge_2_coloring_comm(complete_graph[V], color, x, z)
        edge_2_coloring_comm(complete_graph[V], color, y, z)
        complete_graph_adj_iff_ne[V](x, y)
        complete_graph[V].adj(x, y) = (x != y)
        complete_graph_adj_iff_ne[V](x, z)
        complete_graph[V].adj(x, z) = (x != z)
        complete_graph_adj_iff_ne[V](y, z)
        complete_graph[V].adj(y, z) = (y != z)
        three_list_contains_cases(x, y, z, u)
        u = x or u = y or u = z
        three_list_contains_cases(x, y, z, v)
        v = x or v = y or v = z
        if u = x {
            if v = x {
                u = v
                u = v or color(u, v)
            } else {
                if v = y {
                    color(x, y)
                    color(u, v)
                    u = v or color(u, v)
                } else {
                    v = z
                    color(x, z)
                    color(u, v)
                    u = v or color(u, v)
                }
                u = v or color(u, v)
            }
            u = v or color(u, v)
        } else {
            if u = y {
                if v = x {
                    x != y
                    complete_graph[V].adj(x, y)
                    edge_2_coloring_comm(complete_graph[V], color, x, y)
                    color(x, y) = color(y, x)
                    color(y, x)
                    color(u, v)
                    u = v or color(u, v)
                } else {
                    if v = y {
                        u = v
                        u = v or color(u, v)
                    } else {
                        v = z
                        color(y, z)
                        color(u, v)
                        u = v or color(u, v)
                    }
                    u = v or color(u, v)
                }
                u = v or color(u, v)
            } else {
                u = z
                if v = x {
                    x != z
                    complete_graph[V].adj(x, z)
                    edge_2_coloring_comm(complete_graph[V], color, x, z)
                    color(x, z) = color(z, x)
                    color(z, x)
                    color(u, v)
                    u = v or color(u, v)
                } else {
                    if v = y {
                        y != z
                        complete_graph[V].adj(y, z)
                        edge_2_coloring_comm(complete_graph[V], color, y, z)
                        color(y, z) = color(z, y)
                        color(z, y)
                        color(u, v)
                        u = v or color(u, v)
                    } else {
                        v = z
                        u = v
                        u = v or color(u, v)
                    }
                    u = v or color(u, v)
                }
                u = v or color(u, v)
            }
            u = v or color(u, v)
        }
        u = v or color(u, v)
    }
}

/// Two members of the blue triangle's witness list are the same vertex or
/// blue-joined, given that both lie in the list.
theorem blue_pairwise_three_list[V](color: (V, V) -> Bool, x: V, y: V, z: V, u: V, v: V) {
    is_edge_2_coloring(complete_graph[V], color) and
    not color(x, y) and not color(x, z) and not color(y, z) and
    List.cons(x, List.cons(y, List.cons(z, List.nil[V]))).contains(u) and
    List.cons(x, List.cons(y, List.cons(z, List.nil[V]))).contains(v)
    implies (u = v or not color(u, v))
} by {
    if is_edge_2_coloring(complete_graph[V], color) and
        not color(x, y) and not color(x, z) and not color(y, z) and
        List.cons(x, List.cons(y, List.cons(z, List.nil[V]))).contains(u) and
        List.cons(x, List.cons(y, List.cons(z, List.nil[V]))).contains(v) {
        edge_2_coloring_comm(complete_graph[V], color, x, y)
        edge_2_coloring_comm(complete_graph[V], color, x, z)
        edge_2_coloring_comm(complete_graph[V], color, y, z)
        complete_graph_adj_iff_ne[V](x, y)
        complete_graph[V].adj(x, y) = (x != y)
        complete_graph_adj_iff_ne[V](x, z)
        complete_graph[V].adj(x, z) = (x != z)
        complete_graph_adj_iff_ne[V](y, z)
        complete_graph[V].adj(y, z) = (y != z)
        three_list_contains_cases(x, y, z, u)
        u = x or u = y or u = z
        three_list_contains_cases(x, y, z, v)
        v = x or v = y or v = z
        if u = x {
            if v = x {
                u = v
                u = v or not color(u, v)
            } else {
                if v = y {
                    not color(x, y)
                    not color(u, v)
                    u = v or not color(u, v)
                } else {
                    v = z
                    not color(x, z)
                    not color(u, v)
                    u = v or not color(u, v)
                }
                u = v or not color(u, v)
            }
            u = v or not color(u, v)
        } else {
            if u = y {
                if v = x {
                    x != y
                    complete_graph[V].adj(x, y)
                    edge_2_coloring_comm(complete_graph[V], color, x, y)
                    color(x, y) = color(y, x)
                    not color(y, x)
                    not color(u, v)
                    u = v or not color(u, v)
                } else {
                    if v = y {
                        u = v
                        u = v or not color(u, v)
                    } else {
                        v = z
                        not color(y, z)
                        not color(u, v)
                        u = v or not color(u, v)
                    }
                    u = v or not color(u, v)
                }
                u = v or not color(u, v)
            } else {
                u = z
                if v = x {
                    x != z
                    complete_graph[V].adj(x, z)
                    edge_2_coloring_comm(complete_graph[V], color, x, z)
                    color(x, z) = color(z, x)
                    not color(z, x)
                    not color(u, v)
                    u = v or not color(u, v)
                } else {
                    if v = y {
                        y != z
                        complete_graph[V].adj(y, z)
                        edge_2_coloring_comm(complete_graph[V], color, y, z)
                        color(y, z) = color(z, y)
                        not color(z, y)
                        not color(u, v)
                        u = v or not color(u, v)
                    } else {
                        v = z
                        u = v
                        u = v or not color(u, v)
                    }
                    u = v or not color(u, v)
                }
                u = v or not color(u, v)
            }
            u = v or not color(u, v)
        }
        u = v or not color(u, v)
    }
}

/// A red triangle in `s` is a red clique of size three in `s`.
theorem red_triangle_to_clique3[V](color: (V, V) -> Bool, s: FiniteSet[V]) {
    is_edge_2_coloring(complete_graph[V], color) and has_red_triangle(color, s)
    implies has_red_clique(color, s, Nat.3)
} by {
    if is_edge_2_coloring(complete_graph[V], color) and has_red_triangle(color, s) {
        has_red_triangle(color, s) = exists(a: V, b: V, c: V) {
            s.contains(a) and s.contains(b) and s.contains(c) and a != b and a != c and b != c
            and color(a, b) and color(a, c) and color(b, c)
        }
        let (x: V, y: V, z: V) satisfy {
            s.contains(x) and s.contains(y) and s.contains(z) and x != y and x != z and y != z
            and color(x, y) and color(x, z) and color(y, z)
        }
        three_list_length(x, y, z)
        List.cons(x, List.cons(y, List.cons(z, List.nil[V]))).length = Nat.3
        three_list_unique(x, y, z)
        List.cons(x, List.cons(y, List.cons(z, List.nil[V]))).is_unique
        forall(u: V) {
            if List.cons(x, List.cons(y, List.cons(z, List.nil[V]))).contains(u) {
                three_list_contains_cases(x, y, z, u)
                u = x or u = y or u = z
                if u = x {
                    s.contains(u)
                } else {
                    if u = y {
                        s.contains(u)
                    } else {
                        u = z
                        s.contains(u)
                    }
                    s.contains(u)
                }
                s.contains(u)
            }
        }
        forall(u: V) {
            List.cons(x, List.cons(y, List.cons(z, List.nil[V]))).contains(u) implies s.contains(u)
        }
        forall(u: V, v: V) {
            if List.cons(x, List.cons(y, List.cons(z, List.nil[V]))).contains(u)
                and List.cons(x, List.cons(y, List.cons(z, List.nil[V]))).contains(v) {
                red_pairwise_three_list(color, x, y, z, u, v)
                u = v or color(u, v)
            }
        }
        forall(u: V, v: V) {
            List.cons(x, List.cons(y, List.cons(z, List.nil[V]))).contains(u) and List.cons(x, List.cons(y, List.cons(z, List.nil[V]))).contains(v) implies (u = v or color(u, v))
        }
        has_red_clique(color, s, Nat.3) = exists(xs: List[V]) {
            xs.length = Nat.3 and xs.is_unique and clique_members(xs, s) and clique_pairwise_red(color, xs)
        }
        clique_members(List.cons(x, List.cons(y, List.cons(z, List.nil[V]))), s)
        clique_pairwise_red(color, List.cons(x, List.cons(y, List.cons(z, List.nil[V]))))
        (List.cons(x, List.cons(y, List.cons(z, List.nil[V]))).length = Nat.3
            and List.cons(x, List.cons(y, List.cons(z, List.nil[V]))).is_unique
            and clique_members(List.cons(x, List.cons(y, List.cons(z, List.nil[V]))), s)
            and clique_pairwise_red(color, List.cons(x, List.cons(y, List.cons(z, List.nil[V])))))
        exists_intro(function(xs: List[V]) {
            xs.length = Nat.3 and xs.is_unique and clique_members(xs, s) and clique_pairwise_red(color, xs)
        }, List.cons(x, List.cons(y, List.cons(z, List.nil[V]))))
        exists(xs: List[V]) {
            xs.length = Nat.3 and xs.is_unique and clique_members(xs, s) and clique_pairwise_red(color, xs)
        }
        has_red_clique(color, s, Nat.3)
    }
}

/// A blue triangle in `s` is a blue clique of size three in `s`.
theorem blue_triangle_to_clique3[V](color: (V, V) -> Bool, s: FiniteSet[V]) {
    is_edge_2_coloring(complete_graph[V], color) and has_blue_triangle(color, s)
    implies has_blue_clique(color, s, Nat.3)
} by {
    if is_edge_2_coloring(complete_graph[V], color) and has_blue_triangle(color, s) {
        has_blue_triangle(color, s) = exists(a: V, b: V, c: V) {
            s.contains(a) and s.contains(b) and s.contains(c) and a != b and a != c and b != c
            and not color(a, b) and not color(a, c) and not color(b, c)
        }
        let (x: V, y: V, z: V) satisfy {
            s.contains(x) and s.contains(y) and s.contains(z) and x != y and x != z and y != z
            and not color(x, y) and not color(x, z) and not color(y, z)
        }
        three_list_length(x, y, z)
        List.cons(x, List.cons(y, List.cons(z, List.nil[V]))).length = Nat.3
        three_list_unique(x, y, z)
        List.cons(x, List.cons(y, List.cons(z, List.nil[V]))).is_unique
        forall(u: V) {
            if List.cons(x, List.cons(y, List.cons(z, List.nil[V]))).contains(u) {
                three_list_contains_cases(x, y, z, u)
                u = x or u = y or u = z
                if u = x {
                    s.contains(u)
                } else {
                    if u = y {
                        s.contains(u)
                    } else {
                        u = z
                        s.contains(u)
                    }
                    s.contains(u)
                }
                s.contains(u)
            }
        }
        forall(u: V) {
            List.cons(x, List.cons(y, List.cons(z, List.nil[V]))).contains(u) implies s.contains(u)
        }
        forall(u: V, v: V) {
            if List.cons(x, List.cons(y, List.cons(z, List.nil[V]))).contains(u)
                and List.cons(x, List.cons(y, List.cons(z, List.nil[V]))).contains(v) {
                blue_pairwise_three_list(color, x, y, z, u, v)
                u = v or not color(u, v)
            }
        }
        forall(u: V, v: V) {
            List.cons(x, List.cons(y, List.cons(z, List.nil[V]))).contains(u) and List.cons(x, List.cons(y, List.cons(z, List.nil[V]))).contains(v) implies (u = v or not color(u, v))
        }
        has_blue_clique(color, s, Nat.3) = exists(xs: List[V]) {
            xs.length = Nat.3 and xs.is_unique and clique_members(xs, s) and clique_pairwise_blue(color, xs)
        }
        clique_members(List.cons(x, List.cons(y, List.cons(z, List.nil[V]))), s)
        clique_pairwise_blue(color, List.cons(x, List.cons(y, List.cons(z, List.nil[V]))))
        (List.cons(x, List.cons(y, List.cons(z, List.nil[V]))).length = Nat.3
            and List.cons(x, List.cons(y, List.cons(z, List.nil[V]))).is_unique
            and clique_members(List.cons(x, List.cons(y, List.cons(z, List.nil[V]))), s)
            and clique_pairwise_blue(color, List.cons(x, List.cons(y, List.cons(z, List.nil[V])))))
        exists_intro(function(xs: List[V]) {
            xs.length = Nat.3 and xs.is_unique and clique_members(xs, s) and clique_pairwise_blue(color, xs)
        }, List.cons(x, List.cons(y, List.cons(z, List.nil[V]))))
        exists(xs: List[V]) {
            xs.length = Nat.3 and xs.is_unique and clique_members(xs, s) and clique_pairwise_blue(color, xs)
        }
        has_blue_clique(color, s, Nat.3)
    }
}

/// R(3, 3) <= 6 in the framework of this file: every 2-coloring of the
/// complete graph on six vertices has a red clique of size three or a blue
/// clique of size three.
theorem ramsey_r33_upper_number {
    ramsey_number_upper(Nat.3, Nat.3, Nat.6)
} by {
    ramsey_number_upper(Nat.3, Nat.3, Nat.6) = forall(color: (Nat, Nat) -> Bool) {
        is_edge_2_coloring(complete_graph[Nat], color) implies (has_red_clique(color, range_set(Nat.6), Nat.3) or has_blue_clique(color, range_set(Nat.6), Nat.3))
    }
    forall(color: (Nat, Nat) -> Bool) {
        if is_edge_2_coloring(complete_graph[Nat], color) {
            ramsey_r33_upper(color)
            has_mono_triangle(color, six_vertices)
            has_mono_triangle(color, six_vertices) =
                (has_red_triangle(color, six_vertices) or has_blue_triangle(color, six_vertices))
            has_red_triangle(color, six_vertices) or has_blue_triangle(color, six_vertices)
            if has_red_triangle(color, six_vertices) {
                red_triangle_to_clique3(color, six_vertices)
                has_red_clique(color, six_vertices, Nat.3)
                has_red_clique(color, six_vertices, Nat.3) or has_blue_clique(color, six_vertices, Nat.3)
            }
            if has_blue_triangle(color, six_vertices) {
                blue_triangle_to_clique3(color, six_vertices)
                has_blue_clique(color, six_vertices, Nat.3)
                has_red_clique(color, six_vertices, Nat.3) or has_blue_clique(color, six_vertices, Nat.3)
            }
            has_red_clique(color, six_vertices, Nat.3) or has_blue_clique(color, six_vertices, Nat.3)
            six_vertices = range_set(Nat.6)
            has_red_clique(color, six_vertices, Nat.3) = has_red_clique(color, range_set(Nat.6), Nat.3)
            has_blue_clique(color, six_vertices, Nat.3) = has_blue_clique(color, range_set(Nat.6), Nat.3)
            has_red_clique(color, range_set(Nat.6), Nat.3) or has_blue_clique(color, range_set(Nat.6), Nat.3)
        }
    }
    ramsey_number_upper(Nat.3, Nat.3, Nat.6)
}

/// The library's R(3, 3) <= 6, restated verbatim in this file.
///
/// `six_vertices` is the six-element set {0, ..., 5}; the conclusion says that
/// some triangle of it is monochromatic.
theorem ramsey_r33_upper_restated(color: (Nat, Nat) -> Bool) {
    is_edge_2_coloring(complete_graph[Nat], color)
    implies has_mono_triangle(color, six_vertices)
} by {
    if is_edge_2_coloring(complete_graph[Nat], color) {
        ramsey_r33_upper(color)
        has_mono_triangle(color, six_vertices)
    }
}

// ============================================================================
// The Erdős–Szekeres theorem, small case r = s = 3.
//
// The general theorem — more than r * s distinct elements of a linear order
// contain an increasing subsequence of length r + 1 or a decreasing
// subsequence of length s + 1 — is proved in
// combinatorics/erdos_szekeres.ac (that module is private to its package, so
// it is not imported here).  This section records the statement of the general
// theorem as a comment and verifies the first nontrivial case (the elementary
// list-index lemmas used below live in the helper module
// ramsey_numbers_esz.ac):
//
//     any five distinct natural numbers contain an increasing subsequence of
//     length three or a decreasing subsequence of length three
//
// (r = s = 3 with (r - 1)(s - 1) + 1 = 5 terms, in the numbering of the
// general theorem with r = s = 2 and r + 1 = s + 1 = 3).
//
// Proof (the classical Erdős–Szekeres argument specialized to 5 terms): to
// each position i assign the tag (up(i), dn(i)), where up(i) says a later term
// is larger and dn(i) says a later term is smaller.  If there is no increasing
// triple and no decreasing triple, then up(i) and dn(i) cannot both fail to
// hold in a way that makes a collision: whenever i < j with x_i < x_j, up(i)
// holds (witness j) while up(j) cannot hold (a later witness k would give the
// increasing triple i < j < k), and symmetrically for decreasing pairs with
// dn.  Since the five positions carry only four tags, two positions i < j
// share a tag; the values at i and j are distinct, so x_i < x_j or x_j < x_i,
// and either case contradicts the shared tag.
// ============================================================================

/// The value at position `i` is strictly less than the value at position `j`.
define lt_at(xs: List[Nat], i: Nat, j: Nat) -> Bool {
    exists(x: Nat, y: Nat) {
        value_at(xs, i, x) and value_at(xs, j, y) and x < y
    }
}

/// The value at position `i` is strictly greater than the value at position `j`.
define gt_at(xs: List[Nat], i: Nat, j: Nat) -> Bool {
    lt_at(xs, j, i)
}

/// Three in-bounds positions with pairwise increasing values.
define esz_inc3(xs: List[Nat]) -> Bool {
    exists(i: Nat, j: Nat, k: Nat) {
        i < j and j < k and k < xs.length and lt_at(xs, i, j) and lt_at(xs, j, k) and lt_at(xs, i, k)
    }
}

/// Three in-bounds positions with pairwise decreasing values.
define esz_dec3(xs: List[Nat]) -> Bool {
    exists(i: Nat, j: Nat, k: Nat) {
        i < j and j < k and k < xs.length and gt_at(xs, i, j) and gt_at(xs, j, k) and gt_at(xs, i, k)
    }
}

/// A later term is larger than the term at position `i`.
define esz_up(xs: List[Nat], i: Nat) -> Bool {
    exists(j: Nat) {
        i < j and j < xs.length and lt_at(xs, i, j)
    }
}

/// A later term is smaller than the term at position `i`.
define esz_dn(xs: List[Nat], i: Nat) -> Bool {
    exists(j: Nat) {
        i < j and j < xs.length and gt_at(xs, i, j)
    }
}

/// The Erdős–Szekeres tag of a position: (a later term is larger, a later term is smaller).
define esz_tag(xs: List[Nat], i: Nat) -> Pair[Bool, Bool] {
    Pair.new(esz_up(xs, i), esz_dn(xs, i))
}

/// The map from positions to their Erdős–Szekeres tags.
define esz_tag_fn(xs: List[Nat], i: Nat) -> Pair[Bool, Bool] {
    esz_tag(xs, i)
}

/// The value at a position is determined uniquely.
theorem value_at_det(xs: List[Nat], i: Nat, x: Nat, y: Nat) {
    value_at(xs, i, x) and value_at(xs, i, y) implies x = y
} by {
    if value_at(xs, i, x) and value_at(xs, i, y) {
        xs.get_idx(i) = Option.some(x)
        xs.get_idx(i) = Option.some(y)
        Option.some(x) = Option.some(y)
        some_injective(x, y)
        x = y
    }
}

/// Comparable values witness the comparison of positions.
theorem lt_at_of_values(xs: List[Nat], i: Nat, j: Nat, x: Nat, y: Nat) {
    value_at(xs, i, x) and value_at(xs, j, y) and x < y implies lt_at(xs, i, j)
}

/// The comparison of positions is transitive.
theorem lt_at_trans(xs: List[Nat], i: Nat, j: Nat, m: Nat) {
    lt_at(xs, i, j) and lt_at(xs, j, m) implies lt_at(xs, i, m)
} by {
    if lt_at(xs, i, j) and lt_at(xs, j, m) {
        let (x: Nat, y: Nat) satisfy {
            value_at(xs, i, x) and value_at(xs, j, y) and x < y
        }
        let (y2: Nat, z: Nat) satisfy {
            value_at(xs, j, y2) and value_at(xs, m, z) and y2 < z
        }
        value_at_det(xs, j, y, y2)
        y = y2
        y < z
        x < y and y < z
        lt_trans(x, y, z)
        x < z
        lt_at_of_values(xs, i, m, x, z)
        lt_at(xs, i, m)
    }
}

/// Every in-bounds position carries a value.
theorem get_idx_some_of_lt(xs: List[Nat], i: Nat) {
    i < xs.length implies exists(x: Nat) {
        xs.get_idx(i) = Option.some(x)
    }
} by {
    esz_get_idx_prop(List.nil[Nat])
    forall(head: Nat, tail: List[Nat]) {
        if esz_get_idx_prop(tail) {
            esz_get_idx_prop_step(head, tail)
            esz_get_idx_prop(List.cons(head, tail))
        }
    }
    esz_get_idx_prop(xs)
    esz_get_idx_prop(xs) = forall(idx: Nat) {
        idx < xs.length implies exists(x: Nat) {
            xs.get_idx(idx) = Option.some(x)
        }
    }
    forall(idx: Nat) {
        idx < xs.length implies exists(x: Nat) {
            xs.get_idx(idx) = Option.some(x)
        }
    }
    i < xs.length implies exists(x: Nat) {
        xs.get_idx(i) = Option.some(x)
    }
}

/// A position that carries a value witnesses containment.
theorem value_at_imp_contains(xs: List[Nat], i: Nat, x: Nat) {
    value_at(xs, i, x) implies xs.contains(x)
} by {
    forall(idx: Nat, v: Nat) {
        if value_at(List.nil[Nat], idx, v) {
            List.nil[Nat].get_idx(idx) = Option.none
            Option.none = Option.some(v)
            false
        }
    }
    esz_value_at_prop(List.nil[Nat])
    forall(head: Nat, tail: List[Nat]) {
        if esz_value_at_prop(tail) {
            esz_value_at_prop_step(head, tail)
            esz_value_at_prop(List.cons(head, tail))
        }
    }
    esz_value_at_prop(xs)
    esz_value_at_prop(xs) = forall(idx: Nat, v: Nat) {
        value_at(xs, idx, v) implies xs.contains(v)
    }
    forall(idx: Nat, v: Nat) {
        value_at(xs, idx, v) implies xs.contains(v)
    }
    value_at(xs, i, x) implies xs.contains(x)
}

/// A position that carries a value counts at least one.
theorem count_ge_one_of_value_at(xs: List[Nat], i: Nat, x: Nat) {
    value_at(xs, i, x) implies xs.count(x) >= Nat.1
} by {
    if value_at(xs, i, x) {
        value_at_imp_contains(xs, i, x)
        xs.contains(x)
        list_contains_implies_count_geq_one(xs, x)
        xs.count(x) >= Nat.1
    }
}

/// Two distinct positions carrying the same value force the count to be at least two.
theorem count_ge_two_of_two_positions(xs: List[Nat], i: Nat, j: Nat, x: Nat) {
    value_at(xs, i, x) and value_at(xs, j, x) and i != j implies xs.count(x) >= Nat.2
} by {
    define p(l: List[Nat]) -> Bool {
        forall(a: Nat, b: Nat) {
            value_at(l, a, x) and value_at(l, b, x) and a != b implies l.count(x) >= Nat.2
        }
    }
    forall(a: Nat, b: Nat) {
        if value_at(List.nil[Nat], a, x) and value_at(List.nil[Nat], b, x) and a != b {
            List.nil[Nat].get_idx(a) = Option.none
            Option.none = Option.some(x)
            false
        }
    }
    p(List.nil[Nat])
    forall(head: Nat, tail: List[Nat]) {
        if p(tail) {
            forall(a: Nat, b: Nat) {
                if value_at(List.cons(head, tail), a, x) and value_at(List.cons(head, tail), b, x) and a != b {
                    if head = x {
                        zero_or_suc(a)
                        if a = Nat.0 {
                            a != b
                            b != Nat.0
                            zero_or_suc(b)
                            let (b0: Nat) satisfy { b = b0.suc }
                            value_at_cons_suc(head, tail, b0, x)
                            value_at(tail, b0, x)
                            count_ge_one_of_value_at(tail, b0, x)
                            tail.count(x) >= Nat.1
                        }
                        if a != Nat.0 {
                            let (a0: Nat) satisfy { a = a0.suc }
                            value_at_cons_suc(head, tail, a0, x)
                            value_at(tail, a0, x)
                            count_ge_one_of_value_at(tail, a0, x)
                            tail.count(x) >= Nat.1
                        }
                        tail.count(x) >= Nat.1
                        List.cons(head, tail).count(x) = 1 + tail.count(x)
                        lte_add_left(Nat.1, Nat.1, tail.count(x))
                        Nat.1 + Nat.1 <= Nat.1 + tail.count(x)
                        List.cons(head, tail).count(x) >= Nat.2
                    }
                    if head != x {
                        if a = Nat.0 {
                            value_at_cons_zero(head, tail, x)
                            head = x
                            head != x
                            false
                        }
                        a != Nat.0
                        zero_or_suc(a)
                        let (a0: Nat) satisfy { a = a0.suc }
                        value_at_cons_suc(head, tail, a0, x)
                        value_at(tail, a0, x)
                        if b = Nat.0 {
                            value_at_cons_zero(head, tail, x)
                            head = x
                            head != x
                            false
                        }
                        b != Nat.0
                        zero_or_suc(b)
                        let (b0: Nat) satisfy { b = b0.suc }
                        if a0 = b0 {
                            a = a0.suc
                            b = b0.suc
                            a = b
                            a != b
                            false
                        }
                        a0 != b0
                        value_at_cons_suc(head, tail, b0, x)
                        value_at(tail, b0, x)
                        (value_at(tail, a0, x) and value_at(tail, b0, x) and a0 != b0 implies tail.count(x) >= Nat.2)
                        value_at(tail, a0, x) and value_at(tail, b0, x) and a0 != b0
                        tail.count(x) >= Nat.2
                        List.cons(head, tail).count(x) = tail.count(x)
                        List.cons(head, tail).count(x) >= Nat.2
                    }
                    List.cons(head, tail).count(x) >= Nat.2
                }
            }
            p(List.cons(head, tail))
        }
    }
    p(xs)
}

/// A unique list carries distinct values at distinct positions.
theorem is_unique_distinct_values(xs: List[Nat], i: Nat, j: Nat) {
    xs.is_unique and i < xs.length and j < xs.length and i != j implies
    not exists(x: Nat) { value_at(xs, i, x) and value_at(xs, j, x) }
} by {
    if xs.is_unique and i < xs.length and j < xs.length and i != j {
        if exists(x: Nat) { value_at(xs, i, x) and value_at(xs, j, x) } {
            let (x: Nat) satisfy { value_at(xs, i, x) and value_at(xs, j, x) }
            count_ge_two_of_two_positions(xs, i, j, x)
            value_at(xs, i, x) and value_at(xs, j, x) and i != j
            xs.count(x) >= Nat.2
            unique_implies_no_duplicate(xs, x)
            xs.count(x) <= Nat.1
            Nat.2 <= xs.count(x)
            lt_suc(Nat.1)
            Nat.1 < Nat.2
            lte_and_lt(xs.count(x), Nat.1, Nat.2)
            xs.count(x) < Nat.2
            lte_and_lt(Nat.2, xs.count(x), Nat.2)
            Nat.2 < Nat.2
            lt_not_ref(Nat.2)
            false
        }
        not exists(x: Nat) { value_at(xs, i, x) and value_at(xs, j, x) }
    }
}

/// Distinct in-bounds positions are strictly comparable one way or the other.
theorem lt_at_total(xs: List[Nat], i: Nat, j: Nat) {
    xs.is_unique and i < j and j < xs.length implies lt_at(xs, i, j) or lt_at(xs, j, i)
} by {
    if xs.is_unique and i < j and j < xs.length {
        j <= xs.length
        lt_and_lte(i, j, xs.length)
        i < j and j <= xs.length
        i < xs.length
        get_idx_some_of_lt(xs, i)
        exists(x: Nat) { xs.get_idx(i) = Option.some(x) }
        let (x: Nat) satisfy { xs.get_idx(i) = Option.some(x) }
        get_idx_some_of_lt(xs, j)
        exists(y: Nat) { xs.get_idx(j) = Option.some(y) }
        let (y: Nat) satisfy { xs.get_idx(j) = Option.some(y) }
        value_at(xs, i, x)
        value_at(xs, j, y)
        if x = y {
            is_unique_distinct_values(xs, i, j)
            exists(z: Nat) { value_at(xs, i, z) and value_at(xs, j, z) }
            false
        }
        x != y
        lte_or_lte_swap(x, y)
        x <= y or y <= x
        if x <= y {
            if x = y {
                false
            }
            x != y
            x < y
            lt_at_of_values(xs, i, j, x, y)
            lt_at(xs, i, j)
            lt_at(xs, i, j) or lt_at(xs, j, i)
        }
        if y <= x {
            if y = x {
                false
            }
            y != x
            y < x
            lt_at_of_values(xs, j, i, y, x)
            lt_at(xs, j, i)
            lt_at(xs, i, j) or lt_at(xs, j, i)
        }
        lt_at(xs, i, j) or lt_at(xs, j, i)
    }
}

/// The four possible tags, as a list.
let esz_bool_pairs: List[Pair[Bool, Bool]] = List.cons(Pair.new(true, true),
    List.cons(Pair.new(true, false), List.cons(Pair.new(false, true),
        List.cons(Pair.new(false, false), List.nil[Pair[Bool, Bool]]))))

/// The head of the tag list contains the first tag.
theorem esz_bool_pairs_contains_head {
    esz_bool_pairs.contains(Pair.new(true, true))
} by {
    cons_contains_head(Pair.new(true, true), List.cons(Pair.new(true, false), List.cons(Pair.new(false, true), List.cons(Pair.new(false, false), List.nil[Pair[Bool, Bool]]))))
    List.cons(Pair.new(true, true), List.cons(Pair.new(true, false), List.cons(Pair.new(false, true), List.cons(Pair.new(false, false), List.nil[Pair[Bool, Bool]])))).contains(Pair.new(true, true))
    esz_bool_pairs.contains(Pair.new(true, true))
}

/// The second tag lies in the tag list.
theorem esz_bool_pairs_contains_second {
    esz_bool_pairs.contains(Pair.new(true, false))
} by {
    cons_contains_of_tail_contains(Pair.new(true, true), List.cons(Pair.new(true, false), List.cons(Pair.new(false, true), List.cons(Pair.new(false, false), List.nil[Pair[Bool, Bool]]))), Pair.new(true, false))
    cons_contains_head(Pair.new(true, false), List.cons(Pair.new(false, true), List.cons(Pair.new(false, false), List.nil[Pair[Bool, Bool]])))
    List.cons(Pair.new(true, false), List.cons(Pair.new(false, true), List.cons(Pair.new(false, false), List.nil[Pair[Bool, Bool]]))).contains(Pair.new(true, false))
    esz_bool_pairs.contains(Pair.new(true, false))
}

/// The third tag lies in the tag list.
theorem esz_bool_pairs_contains_third {
    esz_bool_pairs.contains(Pair.new(false, true))
} by {
    cons_contains_of_tail_contains(Pair.new(true, true), List.cons(Pair.new(true, false), List.cons(Pair.new(false, true), List.cons(Pair.new(false, false), List.nil[Pair[Bool, Bool]]))), Pair.new(false, true))
    cons_contains_of_tail_contains(Pair.new(true, false), List.cons(Pair.new(false, true), List.cons(Pair.new(false, false), List.nil[Pair[Bool, Bool]])), Pair.new(false, true))
    cons_contains_head(Pair.new(false, true), List.cons(Pair.new(false, false), List.nil[Pair[Bool, Bool]]))
    List.cons(Pair.new(false, true), List.cons(Pair.new(false, false), List.nil[Pair[Bool, Bool]])).contains(Pair.new(false, true))
    esz_bool_pairs.contains(Pair.new(false, true))
}

/// The fourth tag lies in the tag list.
theorem esz_bool_pairs_contains_fourth {
    esz_bool_pairs.contains(Pair.new(false, false))
} by {
    cons_contains_of_tail_contains(Pair.new(true, true), List.cons(Pair.new(true, false), List.cons(Pair.new(false, true), List.cons(Pair.new(false, false), List.nil[Pair[Bool, Bool]]))), Pair.new(false, false))
    cons_contains_of_tail_contains(Pair.new(true, false), List.cons(Pair.new(false, true), List.cons(Pair.new(false, false), List.nil[Pair[Bool, Bool]])), Pair.new(false, false))
    cons_contains_of_tail_contains(Pair.new(false, true), List.cons(Pair.new(false, false), List.nil[Pair[Bool, Bool]]), Pair.new(false, false))
    cons_contains_head(Pair.new(false, false), List.nil[Pair[Bool, Bool]])
    List.cons(Pair.new(false, false), List.nil[Pair[Bool, Bool]]).contains(Pair.new(false, false))
    esz_bool_pairs.contains(Pair.new(false, false))
}

/// A false boolean proposition is equal to `false`.
theorem esz_bool_false_eq(b: Bool) {
    not b implies b = false
} by {
    if not b {
        b = false
    }
}

/// A tag with both components true is the first box entry.
theorem esz_pair_eq_tt(p: Pair[Bool, Bool]) {
    p.first and p.second implies Pair.new(true, true) = p
} by {
    if p.first and p.second {
        pair_new_first(true, true)
        Pair.new(true, true).first = true
        pair_new_second(true, true)
        Pair.new(true, true).second = true
        pair_new_first(p.first, p.second)
        Pair.new(p.first, p.second).first = p.first
        pair_new_second(p.first, p.second)
        Pair.new(p.first, p.second).second = p.second
        Pair.new(true, true).first = Pair.new(p.first, p.second).first
        Pair.new(true, true).second = Pair.new(p.first, p.second).second
        pair_ext(Pair.new(true, true), Pair.new(p.first, p.second))
        Pair.new(true, true) = Pair.new(p.first, p.second)
        pair_eta(p)
        Pair.new(p.first, p.second) = p
        Pair.new(true, true) = p
    }
}

/// A tag with first component true and second false is the second box entry.
theorem esz_pair_eq_tf(p: Pair[Bool, Bool]) {
    p.first and not p.second implies Pair.new(true, false) = p
} by {
    if p.first and not p.second {
        esz_bool_false_eq(p.second)
        p.second = false
        pair_new_first(true, false)
        Pair.new(true, false).first = true
        pair_new_second(true, false)
        Pair.new(true, false).second = false
        pair_new_first(p.first, p.second)
        Pair.new(p.first, p.second).first = p.first
        pair_new_second(p.first, p.second)
        Pair.new(p.first, p.second).second = p.second
        Pair.new(true, false).first = Pair.new(p.first, p.second).first
        Pair.new(true, false).second = Pair.new(p.first, p.second).second
        pair_ext(Pair.new(true, false), Pair.new(p.first, p.second))
        Pair.new(true, false) = Pair.new(p.first, p.second)
        pair_eta(p)
        Pair.new(p.first, p.second) = p
        Pair.new(true, false) = p
    }
}

/// A tag with first component false and second true is the third box entry.
theorem esz_pair_eq_ft(p: Pair[Bool, Bool]) {
    not p.first and p.second implies Pair.new(false, true) = p
} by {
    if not p.first and p.second {
        esz_bool_false_eq(p.first)
        p.first = false
        pair_new_first(false, true)
        Pair.new(false, true).first = false
        pair_new_second(false, true)
        Pair.new(false, true).second = true
        pair_new_first(p.first, p.second)
        Pair.new(p.first, p.second).first = p.first
        pair_new_second(p.first, p.second)
        Pair.new(p.first, p.second).second = p.second
        Pair.new(false, true).first = Pair.new(p.first, p.second).first
        Pair.new(false, true).second = Pair.new(p.first, p.second).second
        pair_ext(Pair.new(false, true), Pair.new(p.first, p.second))
        Pair.new(false, true) = Pair.new(p.first, p.second)
        pair_eta(p)
        Pair.new(p.first, p.second) = p
        Pair.new(false, true) = p
    }
}

/// A tag with both components false is the fourth box entry.
theorem esz_pair_eq_ff(p: Pair[Bool, Bool]) {
    not p.first and not p.second implies Pair.new(false, false) = p
} by {
    if not p.first and not p.second {
        esz_bool_false_eq(p.first)
        p.first = false
        esz_bool_false_eq(p.second)
        p.second = false
        pair_new_first(false, false)
        Pair.new(false, false).first = false
        pair_new_second(false, false)
        Pair.new(false, false).second = false
        pair_new_first(p.first, p.second)
        Pair.new(p.first, p.second).first = p.first
        pair_new_second(p.first, p.second)
        Pair.new(p.first, p.second).second = p.second
        Pair.new(false, false).first = Pair.new(p.first, p.second).first
        Pair.new(false, false).second = Pair.new(p.first, p.second).second
        pair_ext(Pair.new(false, false), Pair.new(p.first, p.second))
        Pair.new(false, false) = Pair.new(p.first, p.second)
        pair_eta(p)
        Pair.new(p.first, p.second) = p
        Pair.new(false, false) = p
    }
}

/// Every tag lies in the list of the four possible tags.
theorem esz_bool_pairs_contains(p: Pair[Bool, Bool]) {
    esz_bool_pairs.contains(p)
} by {
    if p.first {
        if p.second {
            esz_pair_eq_tt(p)
            Pair.new(true, true) = p
            esz_bool_pairs_contains_head
            esz_bool_pairs.contains(Pair.new(true, true))
            esz_bool_pairs.contains(p)
        } else {
            esz_pair_eq_tf(p)
            Pair.new(true, false) = p
            esz_bool_pairs_contains_second
            esz_bool_pairs.contains(Pair.new(true, false))
            esz_bool_pairs.contains(p)
        }
        esz_bool_pairs.contains(p)
    } else {
        if p.second {
            esz_pair_eq_ft(p)
            Pair.new(false, true) = p
            esz_bool_pairs_contains_third
            esz_bool_pairs.contains(Pair.new(false, true))
            esz_bool_pairs.contains(p)
        } else {
            esz_pair_eq_ff(p)
            Pair.new(false, false) = p
            esz_bool_pairs_contains_fourth
            esz_bool_pairs.contains(Pair.new(false, false))
            esz_bool_pairs.contains(p)
        }
        esz_bool_pairs.contains(p)
    }
    esz_bool_pairs.contains(p)
}

/// The length of a prepended list is the successor of the tail's length.
theorem cons_length_generic[T](head: T, tail: List[T]) {
    List.cons(head, tail).length = tail.length.suc
} by {
    List.cons(head, tail).length = tail.length.suc
}

/// The list of the four possible tags has length four.
theorem esz_bool_pairs_length {
    esz_bool_pairs.length = Nat.4
} by {
    List.nil[Pair[Bool, Bool]].length = Nat.0
    cons_length_generic(Pair.new(false, false), List.nil[Pair[Bool, Bool]])
    List.cons(Pair.new(false, false), List.nil[Pair[Bool, Bool]]).length = List.nil[Pair[Bool, Bool]].length.suc
    List.cons(Pair.new(false, false), List.nil[Pair[Bool, Bool]]).length = Nat.0.suc
    Nat.0.suc = Nat.1
    List.cons(Pair.new(false, false), List.nil[Pair[Bool, Bool]]).length = Nat.1
    cons_length_generic(Pair.new(false, true), List.cons(Pair.new(false, false), List.nil[Pair[Bool, Bool]]))
    List.cons(Pair.new(false, true), List.cons(Pair.new(false, false), List.nil[Pair[Bool, Bool]])).length = List.cons(Pair.new(false, false), List.nil[Pair[Bool, Bool]]).length.suc
    List.cons(Pair.new(false, true), List.cons(Pair.new(false, false), List.nil[Pair[Bool, Bool]])).length = Nat.1.suc
    Nat.1.suc = Nat.2
    List.cons(Pair.new(false, true), List.cons(Pair.new(false, false), List.nil[Pair[Bool, Bool]])).length = Nat.2
    cons_length_generic(Pair.new(true, false), List.cons(Pair.new(false, true), List.cons(Pair.new(false, false), List.nil[Pair[Bool, Bool]])))
    List.cons(Pair.new(true, false), List.cons(Pair.new(false, true), List.cons(Pair.new(false, false), List.nil[Pair[Bool, Bool]]))).length = List.cons(Pair.new(false, true), List.cons(Pair.new(false, false), List.nil[Pair[Bool, Bool]])).length.suc
    List.cons(Pair.new(true, false), List.cons(Pair.new(false, true), List.cons(Pair.new(false, false), List.nil[Pair[Bool, Bool]]))).length = Nat.2.suc
    Nat.2.suc = Nat.3
    List.cons(Pair.new(true, false), List.cons(Pair.new(false, true), List.cons(Pair.new(false, false), List.nil[Pair[Bool, Bool]]))).length = Nat.3
    cons_length_generic(Pair.new(true, true), List.cons(Pair.new(true, false), List.cons(Pair.new(false, true), List.cons(Pair.new(false, false), List.nil[Pair[Bool, Bool]]))))
    List.cons(Pair.new(true, true), List.cons(Pair.new(true, false), List.cons(Pair.new(false, true), List.cons(Pair.new(false, false), List.nil[Pair[Bool, Bool]])))).length = List.cons(Pair.new(true, false), List.cons(Pair.new(false, true), List.cons(Pair.new(false, false), List.nil[Pair[Bool, Bool]]))).length.suc
    List.cons(Pair.new(true, true), List.cons(Pair.new(true, false), List.cons(Pair.new(false, true), List.cons(Pair.new(false, false), List.nil[Pair[Bool, Bool]])))).length = Nat.3.suc
    Nat.3.suc = Nat.4
    List.cons(Pair.new(true, true), List.cons(Pair.new(true, false), List.cons(Pair.new(false, true), List.cons(Pair.new(false, false), List.nil[Pair[Bool, Bool]])))).length = Nat.4
    esz_bool_pairs.length = List.cons(Pair.new(true, true), List.cons(Pair.new(true, false), List.cons(Pair.new(false, true), List.cons(Pair.new(false, false), List.nil[Pair[Bool, Bool]])))).length
    esz_bool_pairs.length = Nat.4
}

/// The unique image list is no longer than any list containing the image.
theorem map_unique_length_le_of_contains[T, U](items: List[T], targets: List[U], f: T -> U) {
    forall(x: T) {
        items.contains(x) implies targets.contains(f(x))
    } implies map(items, f).unique.length <= targets.length
} by {
    if forall(x: T) {
        items.contains(x) implies targets.contains(f(x))
    } {
        forall(y: U) {
            if map(items, f).contains(y) {
                map_contains(items, f, y)
                let (x: T) satisfy {
                    items.contains(x) and f(x) = y
                }
                targets.contains(y)
            }
        }
        unique_is_smallest_containing_list(map(items, f), targets)
        map(items, f).unique.length <= targets.length
    }
}

/// The pigeonhole step: five positions cannot carry four distinct tags.
theorem esz33_collision(xs: List[Nat]) {
    xs.is_unique and xs.length = Nat.5 implies
    exists(i: Nat, j: Nat) {
        i < j and j < xs.length and esz_tag(xs, i) = esz_tag(xs, j)
    }
} by {
    if xs.is_unique and xs.length = Nat.5 {
        forall(idx: Nat) {
            esz_bool_pairs_contains(esz_tag(xs, idx))
            esz_bool_pairs.contains(esz_tag(xs, idx))
        }
        forall(x: Nat) {
            if xs.length.range.contains(x) {
                esz_bool_pairs_contains(esz_tag(xs, x))
                esz_bool_pairs.contains(esz_tag(xs, x))
            }
            xs.length.range.contains(x) implies esz_bool_pairs.contains(esz_tag(xs, x))
        }
        map_unique_length_le_of_contains(xs.length.range, esz_bool_pairs, esz_tag_fn(xs))
        map(xs.length.range, esz_tag_fn(xs)).unique.length <= esz_bool_pairs.length
        esz_bool_pairs_length
        esz_bool_pairs.length = Nat.4
        map(xs.length.range, esz_tag_fn(xs)).unique.length <= Nat.4
        Nat.4 < Nat.5
        xs.length = Nat.5
        Nat.4 < xs.length
        lte_and_lt(map(xs.length.range, esz_tag_fn(xs)).unique.length, Nat.4, xs.length)
        map(xs.length.range, esz_tag_fn(xs)).unique.length < xs.length
        if map(xs.length.range, esz_tag_fn(xs)).is_unique {
            map(xs.length.range, esz_tag_fn(xs)).unique.length = map(xs.length.range, esz_tag_fn(xs)).length
            map(xs.length.range, esz_tag_fn(xs)).length = xs.length.range.length
            xs.length.range.length = xs.length
            map(xs.length.range, esz_tag_fn(xs)).unique.length = xs.length
            false
        }
        not map(xs.length.range, esz_tag_fn(xs)).is_unique
        range_is_unique(xs.length)
        pigeonhole_unique_map_in(xs.length.range, esz_tag_fn(xs))
        exists(x: Nat, y: Nat) {
            xs.length.range.contains(x) and xs.length.range.contains(y) and x != y and esz_tag(xs, x) = esz_tag(xs, y)
        }
        let (x: Nat, y: Nat) satisfy {
            xs.length.range.contains(x) and xs.length.range.contains(y) and x != y and esz_tag(xs, x) = esz_tag(xs, y)
        }
        lt_of_range_contains(xs.length, x)
        lt_of_range_contains(xs.length, y)
        x < xs.length
        y < xs.length
        trichotomy(x, y)
        if x < y {
            x < y and y < xs.length and esz_tag(xs, x) = esz_tag(xs, y)
            exists(i: Nat, j: Nat) {
                i < j and j < xs.length and esz_tag(xs, i) = esz_tag(xs, j)
            }
        }
        if y < x {
            y < x and x < xs.length and esz_tag(xs, y) = esz_tag(xs, x)
            exists(i: Nat, j: Nat) {
                i < j and j < xs.length and esz_tag(xs, i) = esz_tag(xs, j)
            }
        }
        exists(i: Nat, j: Nat) {
            i < j and j < xs.length and esz_tag(xs, i) = esz_tag(xs, j)
        }
    }
}

/// A larger later value witnesses an increasing pair from `i`.
theorem esz_up_of_lt(xs: List[Nat], i: Nat, j: Nat) {
    i < j and j < xs.length and lt_at(xs, i, j) implies esz_up(xs, i)
} by {
    if i < j and j < xs.length and lt_at(xs, i, j) {
        esz_up(xs, i) = exists(j2: Nat) {
            i < j2 and j2 < xs.length and lt_at(xs, i, j2)
        }
        exists_intro(function(j2: Nat) {
            i < j2 and j2 < xs.length and lt_at(xs, i, j2)
        }, j)
        i < j and j < xs.length and lt_at(xs, i, j)
        exists(j2: Nat) {
            i < j2 and j2 < xs.length and lt_at(xs, i, j2)
        }
        esz_up(xs, i)
    }
}

/// A smaller later value witnesses a decreasing pair from `i`.
theorem esz_dn_of_gt(xs: List[Nat], i: Nat, j: Nat) {
    i < j and j < xs.length and gt_at(xs, i, j) implies esz_dn(xs, i)
} by {
    if i < j and j < xs.length and gt_at(xs, i, j) {
        esz_dn(xs, i) = exists(j2: Nat) {
            i < j2 and j2 < xs.length and gt_at(xs, i, j2)
        }
        exists_intro(function(j2: Nat) {
            i < j2 and j2 < xs.length and gt_at(xs, i, j2)
        }, j)
        i < j and j < xs.length and gt_at(xs, i, j)
        exists(j2: Nat) {
            i < j2 and j2 < xs.length and gt_at(xs, i, j2)
        }
        esz_dn(xs, i)
    }
}

/// Two increasing pairs on three in-bounds positions form an increasing triple.
theorem esz_inc3_of_lt_pair(xs: List[Nat], i: Nat, j: Nat, k: Nat) {
    i < j and j < k and k < xs.length and lt_at(xs, i, j) and lt_at(xs, j, k)
    implies esz_inc3(xs)
} by {
    if i < j and j < k and k < xs.length and lt_at(xs, i, j) and lt_at(xs, j, k) {
        lt_at_trans(xs, i, j, k)
        lt_at(xs, i, j) and lt_at(xs, j, k)
        lt_at(xs, i, k)
        esz_inc3(xs) = exists(i2: Nat, j2: Nat, k2: Nat) {
            i2 < j2 and j2 < k2 and k2 < xs.length and lt_at(xs, i2, j2) and lt_at(xs, j2, k2) and lt_at(xs, i2, k2)
        }
        exists_intro(function(i2: Nat) {
            exists(j2: Nat, k2: Nat) {
                i2 < j2 and j2 < k2 and k2 < xs.length and lt_at(xs, i2, j2) and lt_at(xs, j2, k2) and lt_at(xs, i2, k2)
            }
        }, i)
        exists_intro(function(j2: Nat) {
            exists(k2: Nat) {
                i < j2 and j2 < k2 and k2 < xs.length and lt_at(xs, i, j2) and lt_at(xs, j2, k2) and lt_at(xs, i, k2)
            }
        }, j)
        exists_intro(function(k2: Nat) {
            i < j and j < k2 and k2 < xs.length and lt_at(xs, i, j) and lt_at(xs, j, k2) and lt_at(xs, i, k2)
        }, k)
        i < j and j < k and k < xs.length and lt_at(xs, i, j) and lt_at(xs, j, k) and lt_at(xs, i, k)
        exists(i2: Nat, j2: Nat, k2: Nat) {
            i2 < j2 and j2 < k2 and k2 < xs.length and lt_at(xs, i2, j2) and lt_at(xs, j2, k2) and lt_at(xs, i2, k2)
        }
        esz_inc3(xs)
    }
}

/// Two decreasing pairs on three in-bounds positions form a decreasing triple.
theorem esz_dec3_of_gt_pair(xs: List[Nat], i: Nat, j: Nat, k: Nat) {
    i < j and j < k and k < xs.length and gt_at(xs, i, j) and gt_at(xs, j, k)
    implies esz_dec3(xs)
} by {
    if i < j and j < k and k < xs.length and gt_at(xs, i, j) and gt_at(xs, j, k) {
        gt_at(xs, i, j) = lt_at(xs, j, i)
        gt_at(xs, j, k) = lt_at(xs, k, j)
        lt_at(xs, j, i)
        lt_at(xs, k, j)
        lt_at_trans(xs, k, j, i)
        lt_at(xs, k, j) and lt_at(xs, j, i)
        lt_at(xs, k, i)
        gt_at(xs, i, k) = lt_at(xs, k, i)
        gt_at(xs, i, k)
        esz_dec3(xs) = exists(i2: Nat, j2: Nat, k2: Nat) {
            i2 < j2 and j2 < k2 and k2 < xs.length and gt_at(xs, i2, j2) and gt_at(xs, j2, k2) and gt_at(xs, i2, k2)
        }
        exists_intro(function(i2: Nat) {
            exists(j2: Nat, k2: Nat) {
                i2 < j2 and j2 < k2 and k2 < xs.length and gt_at(xs, i2, j2) and gt_at(xs, j2, k2) and gt_at(xs, i2, k2)
            }
        }, i)
        exists_intro(function(j2: Nat) {
            exists(k2: Nat) {
                i < j2 and j2 < k2 and k2 < xs.length and gt_at(xs, i, j2) and gt_at(xs, j2, k2) and gt_at(xs, i, k2)
            }
        }, j)
        exists_intro(function(k2: Nat) {
            i < j and j < k2 and k2 < xs.length and gt_at(xs, i, j) and gt_at(xs, j, k2) and gt_at(xs, i, k2)
        }, k)
        i < j and j < k and k < xs.length and gt_at(xs, i, j) and gt_at(xs, j, k) and gt_at(xs, i, k)
        exists(i2: Nat, j2: Nat, k2: Nat) {
            i2 < j2 and j2 < k2 and k2 < xs.length and gt_at(xs, i2, j2) and gt_at(xs, j2, k2) and gt_at(xs, i2, k2)
        }
        esz_dec3(xs)
    }
}

/// Without an increasing triple, a larger later value at `j` is impossible.
theorem esz_no_inc3_not_up(xs: List[Nat], i: Nat, j: Nat) {
    not esz_inc3(xs) and i < j and j < xs.length and lt_at(xs, i, j) implies not esz_up(xs, j)
} by {
    if not esz_inc3(xs) and i < j and j < xs.length and lt_at(xs, i, j) {
        if esz_up(xs, j) {
            esz_up(xs, j) = exists(j2: Nat) {
                j < j2 and j2 < xs.length and lt_at(xs, j, j2)
            }
            let (k: Nat) satisfy {
                j < k and k < xs.length and lt_at(xs, j, k)
            }
            esz_inc3_of_lt_pair(xs, i, j, k)
            i < j and j < k and k < xs.length and lt_at(xs, i, j) and lt_at(xs, j, k)
            esz_inc3(xs)
            false
        }
        not esz_up(xs, j)
    }
}

/// Without a decreasing triple, a smaller later value at `j` is impossible.
theorem esz_no_dec3_not_dn(xs: List[Nat], i: Nat, j: Nat) {
    not esz_dec3(xs) and i < j and j < xs.length and gt_at(xs, i, j) implies not esz_dn(xs, j)
} by {
    if not esz_dec3(xs) and i < j and j < xs.length and gt_at(xs, i, j) {
        if esz_dn(xs, j) {
            esz_dn(xs, j) = exists(j2: Nat) {
                j < j2 and j2 < xs.length and gt_at(xs, j, j2)
            }
            let (k: Nat) satisfy {
                j < k and k < xs.length and gt_at(xs, j, k)
            }
            esz_dec3_of_gt_pair(xs, i, j, k)
            i < j and j < k and k < xs.length and gt_at(xs, i, j) and gt_at(xs, j, k)
            esz_dec3(xs)
            false
        }
        not esz_dn(xs, j)
    }
}

/// The Erdős–Szekeres theorem, r = s = 3: any five distinct natural numbers
/// contain an increasing subsequence of length three or a decreasing
/// subsequence of length three.
theorem erdos_szekeres_three_three(xs: List[Nat]) {
    xs.is_unique and xs.length = Nat.5 implies esz_inc3(xs) or esz_dec3(xs)
} by {
    if xs.is_unique and xs.length = Nat.5 {
        if not (esz_inc3(xs) or esz_dec3(xs)) {
            not esz_inc3(xs)
            not esz_dec3(xs)
            esz33_collision(xs)
            xs.is_unique and xs.length = Nat.5
            exists(i: Nat, j: Nat) {
                i < j and j < xs.length and esz_tag(xs, i) = esz_tag(xs, j)
            }
            let (i: Nat, j: Nat) satisfy {
                i < j and j < xs.length and esz_tag(xs, i) = esz_tag(xs, j)
            }
            i < j
            j < xs.length
            j <= xs.length
            lt_and_lte(i, j, xs.length)
            i < j and j <= xs.length
            i < xs.length
            get_idx_some_of_lt(xs, i)
            exists(x: Nat) { xs.get_idx(i) = Option.some(x) }
            let (x: Nat) satisfy { xs.get_idx(i) = Option.some(x) }
            get_idx_some_of_lt(xs, j)
            exists(y: Nat) { xs.get_idx(j) = Option.some(y) }
            let (y: Nat) satisfy { xs.get_idx(j) = Option.some(y) }
            value_at(xs, i, x)
            value_at(xs, j, y)
            is_unique_distinct_values(xs, i, j)
            i != j
            not exists(z: Nat) { value_at(xs, i, z) and value_at(xs, j, z) }
            if x = y {
                value_at(xs, i, x) and value_at(xs, j, x)
                exists(z: Nat) { value_at(xs, i, z) and value_at(xs, j, z) }
                false
            }
            x != y
            lte_or_lte_swap(x, y)
            x <= y or y <= x
            if x <= y {
                if x = y {
                    false
                }
                x != y
                x < y
                lt_at_of_values(xs, i, j, x, y)
                lt_at(xs, i, j)
                esz_up_of_lt(xs, i, j)
                esz_up(xs, i)
                esz_no_inc3_not_up(xs, i, j)
                not esz_up(xs, j)
                esz_tag(xs, i) = Pair.new(esz_up(xs, i), esz_dn(xs, i))
                esz_tag(xs, j) = Pair.new(esz_up(xs, j), esz_dn(xs, j))
                Pair.new(esz_up(xs, i), esz_dn(xs, i)) = Pair.new(esz_up(xs, j), esz_dn(xs, j))
                Pair.new(esz_up(xs, i), esz_dn(xs, i)).first = esz_up(xs, i)
                Pair.new(esz_up(xs, j), esz_dn(xs, j)).first = esz_up(xs, j)
                esz_up(xs, i) = esz_up(xs, j)
                esz_up(xs, j)
                false
            }
            if y <= x {
                if y = x {
                    false
                }
                y != x
                y < x
                lt_at_of_values(xs, j, i, y, x)
                gt_at(xs, i, j) = lt_at(xs, j, i)
                gt_at(xs, i, j)
                esz_dn_of_gt(xs, i, j)
                esz_dn(xs, i)
                esz_no_dec3_not_dn(xs, i, j)
                not esz_dn(xs, j)
                esz_tag(xs, i) = Pair.new(esz_up(xs, i), esz_dn(xs, i))
                esz_tag(xs, j) = Pair.new(esz_up(xs, j), esz_dn(xs, j))
                Pair.new(esz_up(xs, i), esz_dn(xs, i)) = Pair.new(esz_up(xs, j), esz_dn(xs, j))
                Pair.new(esz_up(xs, i), esz_dn(xs, i)).second = esz_dn(xs, i)
                Pair.new(esz_up(xs, j), esz_dn(xs, j)).second = esz_dn(xs, j)
                esz_dn(xs, i) = esz_dn(xs, j)
                esz_dn(xs, j)
                false
            }
            false
        }
        esz_inc3(xs) or esz_dec3(xs)
    }
}

// ============================================================================
// Statements whose full proofs are not yet verified.
//
// Per the project convention, theorems that are not yet proved are recorded
// here as comments rather than as axioms.  Each comment states the theorem,
// the classical proof, and which ingredient is missing.
// ============================================================================

// ----------------------------------------------------------------------------
// The Ramsey recurrence itself.
//
//     theorem ramsey_recurrence_theorem(k: Nat, l: Nat, a: Nat, b: Nat) {
//         ramsey_recurrence(k, l, a, b)
//     }
//
// The predicate `ramsey_recurrence` is defined above and the arithmetic
// consequences R(3, 5) <= 14 and R(4, 4) <= 18 are verified.  The full proof
// fixes a vertex v of K_{a + b}; among its a + b - 1 neighbours, either a
// share the colour of the edge from v or b are opposite-coloured (pigeonhole
// on the two colours).  In the first case the bound R(k - 1, l) <= a applied
// to those a red neighbours gives a red K_{k - 1} (closing a red K_k with v)
// or a blue K_l; in the second the bound R(k, l - 1) <= b applied to the b
// blue neighbours gives a red K_k or a blue K_{l - 1} (closing a blue K_l
// with v).  The missing ingredient is that `ramsey_number_upper` states the
// sub-bounds over the fixed vertex sets `range_set(a)` and `range_set(b)`, so
// applying them to the neighbourhoods of v requires transporting the
// statement along an order-preserving bijection (the relation-transport
// machinery in data.basic.relation_transport), which is not yet verified.
// ----------------------------------------------------------------------------

// ----------------------------------------------------------------------------
// R(3, 5) <= 14.
//
//     theorem ramsey_r35_upper {
//         ramsey_number_upper(Nat.3, Nat.5, Nat.14)
//     }
//
// The classical proof is the recurrence at (k, l) = (3, 5) with a = 5,
// b = 9, using R(2, 5) <= 5 and R(3, 4) <= 9: R(3, 5) <= 5 + 9 = 14.  The
// arithmetic 5 + 9 = 14 is `ramsey_r35_bound_arith` and the reduction through
// the recurrence is `ramsey_r35_from_recurrence` (verified above, granted the
// two sub-bounds).  The remaining ingredients are the full recurrence proof
// (see above) and the two sub-bounds R(2, 5) <= 5 (a red edge or an all-blue
// K5, an easy case analysis) and R(3, 4) <= 9 in the asymmetric form
// "red K3 or blue K4" (the library's `ramsey_r34_upper` proves the symmetric
// form "monochromatic triangle or monochromatic K4", and the asymmetric
// derivation is not yet recorded).
// ----------------------------------------------------------------------------

// ----------------------------------------------------------------------------
// R(4, 4) <= 18.
//
//     theorem ramsey_r44_upper {
//         ramsey_number_upper(Nat.4, Nat.4, Nat.18)
//     }
//
// The classical proof is the recurrence at (k, l) = (4, 4) with a = b = 9,
// using R(3, 4) <= 9 and R(4, 3) = R(3, 4) <= 9 (the two colours are
// symmetric): R(4, 4) <= 9 + 9 = 18.  The arithmetic 9 + 9 = 18 is
// `ramsey_r44_bound_arith` and the reduction through the recurrence is
// `ramsey_r44_from_recurrence` (verified above, granted the two sub-bounds).
// The remaining ingredients are the full recurrence proof and the asymmetric
// R(3, 4) <= 9 sub-bound, exactly as for R(3, 5) <= 14.  (A direct 18-vertex
// proof via the 17 -> 9 pigeonhole on the edges from a vertex and the
// asymmetric R(3, 4) <= 9 is sketched in ramsey_44.ac.)
// ----------------------------------------------------------------------------

// ----------------------------------------------------------------------------
// R(3, 3) = 6.
//
// The upper bound R(3, 3) <= 6 is verified in two forms:
// `ramsey_r33_upper_restated` (the library's triangle form, verbatim) and
// `ramsey_r33_upper_number` (the clique form `ramsey_number_upper(3, 3, 6)`
// of this file).  The lower bound R(3, 3) > 5 is the C5-colouring witness
// (a 5-cycle in one colour and its complement in the other has no
// monochromatic triangle); the colouring `c5_color` and the verification
// that it is an edge 2-coloring live in graph/ramsey_c5.ac and
// graph/ramsey_lower_bound.ac, but the case analysis that no three vertices
// form a monochromatic triangle is not yet verified there, so the equality
// R(3, 3) = 6 is recorded here only as
//
//     // R(3, 3) = 6: R(3, 3) <= 6 (verified above) and R(3, 3) > 5
//     // (classical C5-colouring witness; lower bound not yet verified).
// ----------------------------------------------------------------------------

// ----------------------------------------------------------------------------
// The general Erdős–Szekeres theorem.
//
//     theorem erdos_szekeres[T: LinearOrder](r: Nat, s: Nat, xs: List[T]) {
//         xs.is_unique and xs.length > r * s implies
//         chain_geq(xs, lt_at(xs), r.suc) or chain_geq(xs, gt_at(xs), s.suc)
//     }
//
// is proved in full generality in combinatorics/erdos_szekeres.ac.  That
// module is private to its package and not re-exported through the
// combinatorics interface, so this file does not import it; instead the small
// case r = s = 3 (five distinct numbers contain an increasing or a decreasing
// subsequence of length three) is re-proved here as
// `erdos_szekeres_three_three` with the list-index helpers in
// ramsey_numbers_esz.ac.
// ----------------------------------------------------------------------------
