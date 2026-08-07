from nat import Nat
from finite_set import FiniteSet
from data.finite.finite_set_card import fs_card
from graph.simple_graph import SimpleGraph, graph_complement
from graph.simple_graph_independent import is_independent_in, is_clique_in
from graph.simple_graph_complement_sets import clique_in_complement_of_independent,
    independent_of_clique_in_complement, independent_in_complement_of_clique,
    clique_of_independent_in_complement
from graph.simple_graph_set_bounds import independence_at_most, independence_at_most_apply,
    independence_at_most_intro, clique_at_most, clique_at_most_apply, clique_at_most_intro

numerals Nat

/// A clique bound on the complement is an independence bound on the graph.
///
/// Independent subsets of a graph are exactly the cliques of its complement, so the
/// two bounds constrain the same family of sets.
theorem independence_at_most_of_complement_clique[V](
    g: SimpleGraph[V], s: FiniteSet[V], n: Nat
) {
    clique_at_most(graph_complement(g), s, n) implies independence_at_most(g, s, n)
} by {
    if clique_at_most(graph_complement(g), s, n) {
        forall(t: FiniteSet[V]) {
            if t.subset_eq(s) and is_independent_in(g, t) {
                clique_in_complement_of_independent(g, t)
                is_clique_in(graph_complement(g), t)
                clique_at_most_apply(graph_complement(g), s, n, t)
                fs_card(t) <= n
            }
            t.subset_eq(s) and is_independent_in(g, t) implies fs_card(t) <= n
        }
        independence_at_most_intro(g, s, n)
        independence_at_most(g, s, n)
    }
}

/// An independence bound on the graph is a clique bound on the complement.
theorem complement_clique_at_most_of_independence[V](
    g: SimpleGraph[V], s: FiniteSet[V], n: Nat
) {
    independence_at_most(g, s, n) implies clique_at_most(graph_complement(g), s, n)
} by {
    if independence_at_most(g, s, n) {
        forall(t: FiniteSet[V]) {
            if t.subset_eq(s) and is_clique_in(graph_complement(g), t) {
                independent_of_clique_in_complement(g, t)
                is_independent_in(g, t)
                independence_at_most_apply(g, s, n, t)
                fs_card(t) <= n
            }
            t.subset_eq(s) and is_clique_in(graph_complement(g), t) implies fs_card(t) <= n
        }
        clique_at_most_intro(graph_complement(g), s, n)
        clique_at_most(graph_complement(g), s, n)
    }
}

/// An independence bound on the complement is a clique bound on the graph.
theorem clique_at_most_of_complement_independence[V](
    g: SimpleGraph[V], s: FiniteSet[V], n: Nat
) {
    independence_at_most(graph_complement(g), s, n) implies clique_at_most(g, s, n)
} by {
    if independence_at_most(graph_complement(g), s, n) {
        forall(t: FiniteSet[V]) {
            if t.subset_eq(s) and is_clique_in(g, t) {
                independent_in_complement_of_clique(g, t)
                is_independent_in(graph_complement(g), t)
                independence_at_most_apply(graph_complement(g), s, n, t)
                fs_card(t) <= n
            }
            t.subset_eq(s) and is_clique_in(g, t) implies fs_card(t) <= n
        }
        clique_at_most_intro(g, s, n)
        clique_at_most(g, s, n)
    }
}

/// A clique bound on the graph is an independence bound on the complement.
theorem complement_independence_at_most_of_clique[V](
    g: SimpleGraph[V], s: FiniteSet[V], n: Nat
) {
    clique_at_most(g, s, n) implies independence_at_most(graph_complement(g), s, n)
} by {
    if clique_at_most(g, s, n) {
        forall(t: FiniteSet[V]) {
            if t.subset_eq(s) and is_independent_in(graph_complement(g), t) {
                clique_of_independent_in_complement(g, t)
                is_clique_in(g, t)
                clique_at_most_apply(g, s, n, t)
                fs_card(t) <= n
            }
            t.subset_eq(s) and is_independent_in(graph_complement(g), t) implies fs_card(t) <= n
        }
        independence_at_most_intro(graph_complement(g), s, n)
        independence_at_most(graph_complement(g), s, n)
    }
}

/// The independence bound of a graph and the clique bound of its complement agree.
theorem independence_at_most_iff_complement_clique[V](
    g: SimpleGraph[V], s: FiniteSet[V], n: Nat
) {
    independence_at_most(g, s, n) = clique_at_most(graph_complement(g), s, n)
} by {
    if independence_at_most(g, s, n) {
        complement_clique_at_most_of_independence(g, s, n)
        clique_at_most(graph_complement(g), s, n)
    }
    if clique_at_most(graph_complement(g), s, n) {
        independence_at_most_of_complement_clique(g, s, n)
        independence_at_most(g, s, n)
    }
}

/// The clique bound of a graph and the independence bound of its complement agree.
theorem clique_at_most_iff_complement_independence[V](
    g: SimpleGraph[V], s: FiniteSet[V], n: Nat
) {
    clique_at_most(g, s, n) = independence_at_most(graph_complement(g), s, n)
} by {
    if clique_at_most(g, s, n) {
        complement_independence_at_most_of_clique(g, s, n)
        independence_at_most(graph_complement(g), s, n)
    }
    if independence_at_most(graph_complement(g), s, n) {
        clique_at_most_of_complement_independence(g, s, n)
        clique_at_most(g, s, n)
    }
}
