from nat import Nat, add_cancels_right, add_comm, mul_two_left
from pair import Pair
from list import List
from data.list.list_filter_count import count_cons_eq_if, count_cons_of_eq, count_cons_other,
    filter_cons_of_true, filter_cons_of_false
from data.list.list_cons_membership import cons_contains_eq, cons_contains_head,
    cons_contains_of_tail_contains, cons_contains_cases, nil_not_contains
from pair import pair_eta, pair_new_first, pair_new_second
from finite_set import FiniteSet, finite_set_sum, finite_set_sum_empty, finite_set_sum_insert,
    finite_set_empty_contains_eq, fs_remove, fs_insert
from data.finite.finite_set_card import fs_card, fs_card_empty
from data.finite.finite_set_membership import fs_remove_contains_eq, fs_insert_contains_eq,
    fs_insert_contains_self, fs_insert_contains_of_contains, finite_set_eq_of_contains_eq
from data.finite.finite_set_card_ops import fs_card_insert_of_not_contains
from data.finite.finite_set_sum_product import finite_set_sum_eq_of_finite_set_eq
from data.finite.finite_set_unique_induction import finite_set_strong_induction
from graph.simple_graph import SimpleGraph, simple_graph_adj_ne, simple_graph_adj_comm
from graph.simple_graph_walks import simple_graph_walk, simple_graph_closed_walk,
    simple_graph_walk_vertices, simple_graph_walk_cons_iff, simple_graph_walk_nil,
    simple_graph_closed_walk_is_walk
from graph.simple_graph_degree import degree, neighborhood, neighborhood_contains_eq,
    degree_eq_neighborhood_card
from graph.simple_graph_edges import directed_edges, directed_edges_contains_pair,
    directed_edges_contains_eq, directed_edges_ne

numerals Nat

/// The constant function on `V` with value zero.
define nat_zero_fn[V](x: V) -> Nat {
    Nat.0
}

/// The constant function on `V` with value one.
define nat_one_fn[V](x: V) -> Nat {
    Nat.1
}

/// One when `x` equals `v`, and zero otherwise.
define vertex_indicator[V](v: V, x: V) -> Nat {
    if x = v { Nat.1 } else { Nat.0 }
}

/// True of the directed edges incident to `v`.
define incident_edge_pred[V](v: V) -> (Pair[V, V] -> Bool) {
    function(p: Pair[V, V]) {
        p.first = v or p.second = v
    }
}

/// The consecutive pairs of a vertex list: one entry per traversed edge.
///
/// `consecutive_pairs_from(prev, xs)` pairs `prev` with the head of `xs` and then
/// proceeds down the list.
define consecutive_pairs_from[V](prev: V, xs: List[V]) -> List[Pair[V, V]] {
    match xs {
        List.nil[V] {
            List.nil[Pair[V, V]]
        }
        List.cons(next, rest) {
            List.cons(Pair.new(prev, next), consecutive_pairs_from(next, rest))
        }
    }
}

/// The consecutive pairs of a vertex list: one entry per traversed edge.
define consecutive_pairs[V](xs: List[V]) -> List[Pair[V, V]] {
    match xs {
        List.nil[V] {
            List.nil[Pair[V, V]]
        }
        List.cons(head, tail) {
            consecutive_pairs_from(head, tail)
        }
    }
}

/// The directed edges traversed by the walk `(start, steps)`, in order.
define walk_edges[V](start: V, steps: List[V]) -> List[Pair[V, V]] {
    consecutive_pairs_from(start, steps)
}

/// The number of times the walk traverses the undirected edge `{x, y}`.
define walk_edge_traversal_count[V](start: V, steps: List[V], x: V, y: V) -> Nat {
    walk_edges(start, steps).count(Pair.new(x, y)) + walk_edges(start, steps).count(Pair.new(y, x))
}

/// The number of edges of the walk incident to `v`.
define walk_edge_visit_count[V](start: V, steps: List[V], v: V) -> Nat {
    walk_edges(start, steps).filter(incident_edge_pred(v)).length
}

/// The summand counting the traversals of `{v, w}` inside a list of directed edges.
define traversal_count_fn[V](es: List[Pair[V, V]], v: V) -> (V -> Nat) {
    function(w: V) {
        es.count(Pair.new(v, w)) + es.count(Pair.new(w, v))
    }
}

/// True when every vertex of the walk lies in `s`.
define walk_vertices_in_set[V](s: FiniteSet[V], start: V, steps: List[V]) -> Bool {
    forall(x: V) {
        simple_graph_walk_vertices(start, steps).contains(x) implies s.contains(x)
    }
}

/// A closed walk of `(g, s)` that traverses every edge of `(g, s)` exactly once.
///
/// The walk is stored as its start vertex and its list of edge targets, matching
/// `simple_graph_walk`. The traversed edges are the consecutive pairs of the walk's
/// vertex list, and `walk_edge_traversal_count` counts how many times an undirected
/// edge `{x, y}` is traversed, in either direction. The circuit is required to stay
/// inside the vertex set `s`, to traverse every edge with both endpoints in `s`
/// exactly once, and (a condition already implied by the first two) to traverse only
/// edges of `(g, s)`; the last conjunct is made explicit so that downstream counting
/// needs no separate derivation.
define is_eulerian_circuit[V](g: SimpleGraph[V], s: FiniteSet[V], start: V, steps: List[V]) -> Bool {
    simple_graph_closed_walk(g, start, steps) and
    walk_vertices_in_set(s, start, steps) and
    forall(p: Pair[V, V]) {
        walk_edges(start, steps).contains(p) implies directed_edges(g, s).contains(p)
    } and
    forall(x: V, y: V) {
        directed_edges(g, s).contains(Pair.new(x, y)) implies
            walk_edge_traversal_count(start, steps, x, y) = Nat.1
    }
}

/// The indicator of an `or` of equalities is the sum of the indicators, when the two
/// candidates are distinct.
theorem vertex_indicator_or[V](v: V, a: V, b: V) {
    a != b implies (if a = v or b = v { Nat.1 } else { Nat.0 }) =
        vertex_indicator(v, a) + vertex_indicator(v, b)
} by {
    if a != b {
        if a = v {
            b != v
            vertex_indicator(v, b) = Nat.0
            vertex_indicator(v, a) = Nat.1
            (if a = v or b = v { Nat.1 } else { Nat.0 }) = Nat.1
            vertex_indicator(v, a) + vertex_indicator(v, b) = Nat.1
            (if a = v or b = v { Nat.1 } else { Nat.0 }) = vertex_indicator(v, a) + vertex_indicator(v, b)
        }
        if a != v {
            if b = v {
                vertex_indicator(v, a) = Nat.0
                vertex_indicator(v, b) = Nat.1
                (if a = v or b = v { Nat.1 } else { Nat.0 }) = Nat.1
                vertex_indicator(v, a) + vertex_indicator(v, b) = Nat.1
                (if a = v or b = v { Nat.1 } else { Nat.0 }) = vertex_indicator(v, a) + vertex_indicator(v, b)
            }
            if b != v {
                vertex_indicator(v, a) = Nat.0
                vertex_indicator(v, b) = Nat.0
                (if a = v or b = v { Nat.1 } else { Nat.0 }) = Nat.0
                vertex_indicator(v, a) + vertex_indicator(v, b) = Nat.0
                (if a = v or b = v { Nat.1 } else { Nat.0 }) = vertex_indicator(v, a) + vertex_indicator(v, b)
            }
            (if a = v or b = v { Nat.1 } else { Nat.0 }) = vertex_indicator(v, a) + vertex_indicator(v, b)
        }
        (if a = v or b = v { Nat.1 } else { Nat.0 }) = vertex_indicator(v, a) + vertex_indicator(v, b)
    }
}

/// Prepending an element adds its indicator to the count of the tail.
theorem vertex_indicator_count_cons[V](v: V, next: V, rest: List[V]) {
    vertex_indicator(v, next) + rest.count(v) = List.cons(next, rest).count(v)
} by {
    if next = v {
        vertex_indicator(v, next) = Nat.1
        count_cons_of_eq(next, rest, v)
        List.cons(next, rest).count(v) = Nat.1 + rest.count(v)
        vertex_indicator(v, next) + rest.count(v) = Nat.1 + rest.count(v)
        vertex_indicator(v, next) + rest.count(v) = List.cons(next, rest).count(v)
    }
    if next != v {
        vertex_indicator(v, next) = Nat.0
        count_cons_other(next, rest, v)
        List.cons(next, rest).count(v) = rest.count(v)
        vertex_indicator(v, next) + rest.count(v) = rest.count(v)
        vertex_indicator(v, next) + rest.count(v) = List.cons(next, rest).count(v)
    }
}

/// The algebra of one induction step: two occurrences of `next` in the visit count
/// become two occurrences of `next` among the targets.
theorem visit_count_algebra[V](u: V, next: V, tail: List[V], a: Nat) {
    a + vertex_indicator(u, next) + (vertex_indicator(u, next) + Nat.2 * tail.count(u)) =
        a + Nat.2 * List.cons(next, tail).count(u)
} by {
    vertex_indicator_count_cons(u, next, tail)
    vertex_indicator(u, next) + tail.count(u) = List.cons(next, tail).count(u)
    mul_two_left(tail.count(u))
    Nat.2 * tail.count(u) = tail.count(u) + tail.count(u)
    mul_two_left(List.cons(next, tail).count(u))
    Nat.2 * List.cons(next, tail).count(u) =
        List.cons(next, tail).count(u) + List.cons(next, tail).count(u)
    a + vertex_indicator(u, next) + (vertex_indicator(u, next) + Nat.2 * tail.count(u)) =
        a + vertex_indicator(u, next) + (vertex_indicator(u, next) + tail.count(u) + tail.count(u))
    a + vertex_indicator(u, next) + (vertex_indicator(u, next) + tail.count(u) + tail.count(u)) =
        a + (vertex_indicator(u, next) + tail.count(u)) +
        (vertex_indicator(u, next) + tail.count(u))
    a + (vertex_indicator(u, next) + tail.count(u)) +
        (vertex_indicator(u, next) + tail.count(u)) =
        a + List.cons(next, tail).count(u) + List.cons(next, tail).count(u)
    a + List.cons(next, tail).count(u) + List.cons(next, tail).count(u) =
        a + Nat.2 * List.cons(next, tail).count(u)
    a + vertex_indicator(u, next) + (vertex_indicator(u, next) + Nat.2 * tail.count(u)) =
        a + Nat.2 * List.cons(next, tail).count(u)
}

/// The list induction principle, packaged as an eliminator.
lemma list_induction_elim[T](p: List[T] -> Bool, items: List[T]) {
    p(List.nil[T]) and forall(head: T, tail: List[T]) { p(tail) implies p(List.cons[T](head, tail)) }
    implies p(items)
} by {
    if p(List.nil[T]) and forall(head: T, tail: List[T]) { p(tail) implies p(List.cons[T](head, tail)) } {
        List.induction(p)
        forall(xs: List[T]) { p(xs) }
        p(items)
    }
}

/// Prepending a vertex to the walk prepends its first edge to the edge list.
theorem walk_edges_cons[V](start: V, next: V, rest: List[V]) {
    walk_edges(start, List.cons(next, rest)) = List.cons(Pair.new(start, next), walk_edges(next, rest))
} by {
    walk_edges(start, List.cons(next, rest)) =
        consecutive_pairs(simple_graph_walk_vertices(start, List.cons(next, rest)))
    simple_graph_walk_vertices(start, List.cons(next, rest)) =
        List.cons(start, List.cons(next, rest))
    consecutive_pairs(List.cons(start, List.cons(next, rest))) =
        consecutive_pairs_from(start, List.cons(next, rest))
    consecutive_pairs_from(start, List.cons(next, rest)) =
        List.cons(Pair.new(start, next), consecutive_pairs_from(next, rest))
    consecutive_pairs_from(next, rest) = consecutive_pairs(simple_graph_walk_vertices(next, rest))
    simple_graph_walk_vertices(next, rest) = List.cons(next, rest)
    consecutive_pairs(List.cons(next, rest)) = consecutive_pairs_from(next, rest)
    walk_edges(next, rest) = consecutive_pairs_from(next, rest)
    List.cons(Pair.new(start, next), consecutive_pairs_from(next, rest)) =
        List.cons(Pair.new(start, next), walk_edges(next, rest))
    walk_edges(start, List.cons(next, rest)) =
        List.cons(Pair.new(start, next), walk_edges(next, rest))
}

/// The empty walk traverses no edges.
theorem walk_edges_nil[V](start: V) {
    walk_edges(start, List.nil[V]) = List.nil[Pair[V, V]]
} by {
    walk_edges(start, List.nil[V]) =
        consecutive_pairs(simple_graph_walk_vertices(start, List.nil[V]))
    simple_graph_walk_vertices(start, List.nil[V]) = List.cons(start, List.nil[V])
    consecutive_pairs(List.cons(start, List.nil[V])) = List.nil[Pair[V, V]]
}

/// The empty walk visits no vertex.
theorem walk_edge_visit_count_nil[V](start: V, v: V) {
    walk_edge_visit_count(start, List.nil[V], v) = Nat.0
} by {
    walk_edges_nil(start)
    walk_edge_visit_count(start, List.nil[V], v) =
        walk_edges(start, List.nil[V]).filter(incident_edge_pred(v)).length
    List.nil[Pair[V, V]].filter(incident_edge_pred(v)) = List.nil[Pair[V, V]]
    List.nil[Pair[V, V]].length = Nat.0
    walk_edge_visit_count(start, List.nil[V], v) = Nat.0
}

/// Prepending a vertex adds its incident first edge to the visit count.
theorem walk_edge_visit_count_cons[V](start: V, next: V, rest: List[V], v: V) {
    walk_edge_visit_count(start, List.cons(next, rest), v) =
        (if start = v or next = v { Nat.1 } else { Nat.0 }) +
        walk_edge_visit_count(next, rest, v)
} by {
    walk_edges_cons(start, next, rest)
    if start = v or next = v {
        incident_edge_pred(v)(Pair.new(start, next)) =
            (Pair.new(start, next).first = v or Pair.new(start, next).second = v)
        Pair.new(start, next).first = start
        Pair.new(start, next).second = next
        incident_edge_pred(v)(Pair.new(start, next)) = (start = v or next = v)
        incident_edge_pred(v)(Pair.new(start, next))
        filter_cons_of_true(Pair.new(start, next), walk_edges(next, rest), incident_edge_pred(v))
        List.cons(Pair.new(start, next), walk_edges(next, rest)).filter(incident_edge_pred(v)) =
            List.cons(Pair.new(start, next), walk_edges(next, rest).filter(incident_edge_pred(v)))
        List.cons(Pair.new(start, next), walk_edges(next, rest)).filter(incident_edge_pred(v)).length =
            walk_edges(next, rest).filter(incident_edge_pred(v)).length.suc
        walk_edge_visit_count(next, rest, v) =
            walk_edges(next, rest).filter(incident_edge_pred(v)).length
        walk_edges(next, rest).filter(incident_edge_pred(v)).length.suc =
            walk_edge_visit_count(next, rest, v) + Nat.1
        walk_edge_visit_count(start, List.cons(next, rest), v) =
            walk_edge_visit_count(next, rest, v) + Nat.1
        (if start = v or next = v { Nat.1 } else { Nat.0 }) = Nat.1
        walk_edge_visit_count(start, List.cons(next, rest), v) =
            (if start = v or next = v { Nat.1 } else { Nat.0 }) + walk_edge_visit_count(next, rest, v)
    }
    if not (start = v or next = v) {
        incident_edge_pred(v)(Pair.new(start, next)) =
            (Pair.new(start, next).first = v or Pair.new(start, next).second = v)
        Pair.new(start, next).first = start
        Pair.new(start, next).second = next
        incident_edge_pred(v)(Pair.new(start, next)) = (start = v or next = v)
        not incident_edge_pred(v)(Pair.new(start, next))
        filter_cons_of_false(Pair.new(start, next), walk_edges(next, rest), incident_edge_pred(v))
        List.cons(Pair.new(start, next), walk_edges(next, rest)).filter(incident_edge_pred(v)) =
            walk_edges(next, rest).filter(incident_edge_pred(v))
        List.cons(Pair.new(start, next), walk_edges(next, rest)).filter(incident_edge_pred(v)).length =
            walk_edge_visit_count(next, rest, v)
        walk_edge_visit_count(start, List.cons(next, rest), v) =
            walk_edge_visit_count(next, rest, v)
        (if start = v or next = v { Nat.1 } else { Nat.0 }) = Nat.0
        walk_edge_visit_count(start, List.cons(next, rest), v) =
            (if start = v or next = v { Nat.1 } else { Nat.0 }) + walk_edge_visit_count(next, rest, v)
    }
}

/// The edges of a walk ending at `close` are the walk's positions; their incident count
/// at `v` is the count of `v` among the targets, doubled, plus the endpoints.
///
/// Stated with `close` on the left so that no truncated subtraction is needed: the
/// closed-walk corollary cancels the endpoint term.
theorem walk_edge_visit_count_identity[V](g: SimpleGraph[V], prev: V, close: V, steps: List[V], v: V) {
    simple_graph_walk(g, prev, close, steps) implies
        walk_edge_visit_count(prev, steps, v) + vertex_indicator(v, close) =
        vertex_indicator(v, prev) + Nat.2 * steps.count(v)
} by {
    define pc(xs: List[V], s: V, t: V, u: V) -> Bool {
        simple_graph_walk(g, s, t, xs) implies
            walk_edge_visit_count(s, xs, u) + vertex_indicator(u, t) =
            vertex_indicator(u, s) + Nat.2 * xs.count(u)
    }

    define p(xs: List[V]) -> Bool {
        forall(s: V, t: V, u: V) {
            pc(xs, s, t, u)
        }
    }

    forall(s: V, t: V, u: V) {
        if simple_graph_walk(g, s, t, List.nil[V]) {
            simple_graph_walk_nil(g, s, t)
            s = t
            walk_edge_visit_count_nil(s, u)
            walk_edge_visit_count(s, List.nil[V], u) = Nat.0
            walk_edge_visit_count(s, List.nil[V], u) + vertex_indicator(u, t) =
                vertex_indicator(u, s)
            vertex_indicator(u, s) + Nat.2 * List.nil[V].count(u) =
                vertex_indicator(u, s)
            walk_edge_visit_count(s, List.nil[V], u) + vertex_indicator(u, t) =
                vertex_indicator(u, s) + Nat.2 * List.nil[V].count(u)
        }
        pc(List.nil[V], s, t, u)
    }
    p(List.nil[V])

    forall(next: V, tail: List[V]) {
        if p(tail) {
            forall(s: V, t: V, u: V) {
                if simple_graph_walk(g, s, t, List.cons(next, tail)) {
                    simple_graph_walk_cons_iff(g, s, t, next, tail)
                    g.adj(s, next)
                    simple_graph_walk(g, next, t, tail)
                    simple_graph_adj_ne(g, s, next)
                    s != next
                    vertex_indicator_or(u, s, next)
                    (if s = u or next = u { Nat.1 } else { Nat.0 }) =
                        vertex_indicator(u, s) + vertex_indicator(u, next)
                    walk_edge_visit_count_cons(s, next, tail, u)
                    walk_edge_visit_count(s, List.cons(next, tail), u) =
                        (if s = u or next = u { Nat.1 } else { Nat.0 }) +
                        walk_edge_visit_count(next, tail, u)
                    walk_edge_visit_count(s, List.cons(next, tail), u) =
                        vertex_indicator(u, s) + vertex_indicator(u, next) +
                        walk_edge_visit_count(next, tail, u)
                    pc(tail, next, t, u)
                    pc(tail, next, t, u) = (
                        simple_graph_walk(g, next, t, tail) implies
                            walk_edge_visit_count(next, tail, u) + vertex_indicator(u, t) =
                            vertex_indicator(u, next) + Nat.2 * tail.count(u)
                    )
                    walk_edge_visit_count(next, tail, u) + vertex_indicator(u, t) =
                        vertex_indicator(u, next) + Nat.2 * tail.count(u)
                    walk_edge_visit_count(s, List.cons(next, tail), u) + vertex_indicator(u, t) =
                        vertex_indicator(u, s) + vertex_indicator(u, next) +
                        walk_edge_visit_count(next, tail, u) + vertex_indicator(u, t)
                    walk_edge_visit_count(s, List.cons(next, tail), u) + vertex_indicator(u, t) =
                        vertex_indicator(u, s) + vertex_indicator(u, next) +
                        (walk_edge_visit_count(next, tail, u) + vertex_indicator(u, t))
                    walk_edge_visit_count(s, List.cons(next, tail), u) + vertex_indicator(u, t) =
                        vertex_indicator(u, s) + vertex_indicator(u, next) +
                        (vertex_indicator(u, next) + Nat.2 * tail.count(u))
                    visit_count_algebra(u, next, tail, vertex_indicator(u, s))
                    vertex_indicator(u, s) + vertex_indicator(u, next) +
                        (vertex_indicator(u, next) + Nat.2 * tail.count(u)) =
                        vertex_indicator(u, s) + Nat.2 * List.cons(next, tail).count(u)
                    walk_edge_visit_count(s, List.cons(next, tail), u) + vertex_indicator(u, t) =
                        vertex_indicator(u, s) + Nat.2 * List.cons(next, tail).count(u)
                }
                pc(List.cons(next, tail), s, t, u)
            }
            p(List.cons(next, tail))
        }
    }
    forall(next: V, tail: List[V]) { p(tail) implies p(List.cons(next, tail)) }
    list_induction_elim(p, steps)
    p(steps)
    pc(steps, prev, close, v)
    if simple_graph_walk(g, prev, close, steps) {
        pc(steps, prev, close, v) = (simple_graph_walk(g, prev, close, steps) implies
            walk_edge_visit_count(prev, steps, v) + vertex_indicator(v, close) =
            vertex_indicator(v, prev) + Nat.2 * steps.count(v))
        walk_edge_visit_count(prev, steps, v) + vertex_indicator(v, close) =
            vertex_indicator(v, prev) + Nat.2 * steps.count(v)
    }
}

/// A closed walk visits each of its edge targets twice: once on the way in, once on the way out.
theorem walk_edge_visit_count_closed[V](g: SimpleGraph[V], start: V, steps: List[V], v: V) {
    simple_graph_closed_walk(g, start, steps) implies
        walk_edge_visit_count(start, steps, v) = Nat.2 * steps.count(v)
} by {
    if simple_graph_closed_walk(g, start, steps) {
        simple_graph_closed_walk_is_walk(g, start, steps)
        simple_graph_walk(g, start, start, steps)
        walk_edge_visit_count_identity(g, start, start, steps, v)
        walk_edge_visit_count(start, steps, v) + vertex_indicator(v, start) =
            vertex_indicator(v, start) + Nat.2 * steps.count(v)
        add_comm(vertex_indicator(v, start), Nat.2 * steps.count(v))
        vertex_indicator(v, start) + Nat.2 * steps.count(v) =
            Nat.2 * steps.count(v) + vertex_indicator(v, start)
        walk_edge_visit_count(start, steps, v) + vertex_indicator(v, start) =
            Nat.2 * steps.count(v) + vertex_indicator(v, start)
        add_cancels_right(walk_edge_visit_count(start, steps, v), Nat.2 * steps.count(v),
            vertex_indicator(v, start))
        walk_edge_visit_count(start, steps, v) = Nat.2 * steps.count(v)
    }
}

/// Finite-set sums agree for functions that agree pointwise on the set.
theorem finite_set_sum_pointwise_eq[V](s: FiniteSet[V], f: V -> Nat, g: V -> Nat) {
    (forall(x: V) { s.contains(x) implies f(x) = g(x) }) implies
        finite_set_sum(s, f) = finite_set_sum(s, g)
} by {
    define p(t: FiniteSet[V]) -> Bool {
        (forall(x: V) { t.contains(x) implies f(x) = g(x) }) implies
            finite_set_sum(t, f) = finite_set_sum(t, g)
    }

    finite_set_sum_empty(f)
    finite_set_sum(FiniteSet.empty[V], f) = Nat.0
    finite_set_sum_empty(g)
    finite_set_sum(FiniteSet.empty[V], g) = Nat.0
    finite_set_sum(FiniteSet.empty[V], f) = finite_set_sum(FiniteSet.empty[V], g)
    p(FiniteSet.empty[V])

    forall(t: FiniteSet[V], item: V) {
        if p(t) and not t.contains(item) {
            if forall(x: V) { fs_insert(t, item).contains(x) implies f(x) = g(x) } {
                fs_insert_contains_self(t, item)
                fs_insert(t, item).contains(item)
                f(item) = g(item)
                forall(x: V) {
                    if t.contains(x) {
                        fs_insert_contains_of_contains(t, item, x)
                        fs_insert(t, item).contains(x)
                        fs_insert(t, item).contains(x) implies f(x) = g(x)
                        f(x) = g(x)
                    }
                    t.contains(x) implies f(x) = g(x)
                }
                p(t)
                p(t) = ((forall(x: V) { t.contains(x) implies f(x) = g(x) }) implies
                    finite_set_sum(t, f) = finite_set_sum(t, g))
                finite_set_sum(t, f) = finite_set_sum(t, g)
                finite_set_sum_insert(t, item, f)
                finite_set_sum(fs_insert(t, item), f) = f(item) + finite_set_sum(t, f)
                finite_set_sum_insert(t, item, g)
                finite_set_sum(fs_insert(t, item), g) = g(item) + finite_set_sum(t, g)
                finite_set_sum(fs_insert(t, item), f) = finite_set_sum(fs_insert(t, item), g)
            }
            p(fs_insert(t, item))
        }
        p(t) and not t.contains(item) implies p(fs_insert(t, item))
    }
    finite_set_strong_induction(p, s)
    p(s)
    p(s) = ((forall(x: V) { s.contains(x) implies f(x) = g(x) }) implies
        finite_set_sum(s, f) = finite_set_sum(s, g))
    if forall(x: V) { s.contains(x) implies f(x) = g(x) } {
        finite_set_sum(s, f) = finite_set_sum(s, g)
    }
}

/// The sum of the zero function is zero.
theorem finite_set_sum_zero[V](s: FiniteSet[V]) {
    finite_set_sum(s, nat_zero_fn[V]) = Nat.0
} by {
    define p(t: FiniteSet[V]) -> Bool {
        finite_set_sum(t, nat_zero_fn[V]) = Nat.0
    }

    finite_set_sum_empty(nat_zero_fn[V])
    finite_set_sum(FiniteSet.empty[V], nat_zero_fn[V]) = Nat.0
    p(FiniteSet.empty[V])

    forall(t: FiniteSet[V], item: V) {
        if p(t) and not t.contains(item) {
            finite_set_sum_insert(t, item, nat_zero_fn[V])
            finite_set_sum(fs_insert(t, item), nat_zero_fn[V]) =
                nat_zero_fn[V](item) + finite_set_sum(t, nat_zero_fn[V])
            nat_zero_fn[V](item) = Nat.0
            finite_set_sum(t, nat_zero_fn[V]) = Nat.0
            nat_zero_fn[V](item) + finite_set_sum(t, nat_zero_fn[V]) = Nat.0
            finite_set_sum(fs_insert(t, item), nat_zero_fn[V]) = Nat.0
        }
        p(t) and not t.contains(item) implies p(fs_insert(t, item))
    }
    finite_set_strong_induction(p, s)
    p(s)
    finite_set_sum(s, nat_zero_fn[V]) = Nat.0
}

/// The sum of the constant-one function is the cardinality of the set.
theorem finite_set_sum_one[V](s: FiniteSet[V]) {
    finite_set_sum(s, nat_one_fn[V]) = fs_card(s)
} by {
    define p(t: FiniteSet[V]) -> Bool {
        finite_set_sum(t, nat_one_fn[V]) = fs_card(t)
    }

    finite_set_sum_empty(nat_one_fn[V])
    finite_set_sum(FiniteSet.empty[V], nat_one_fn[V]) = Nat.0
    fs_card_empty[V]
    fs_card(FiniteSet.empty[V]) = Nat.0
    p(FiniteSet.empty[V])

    forall(t: FiniteSet[V], item: V) {
        if p(t) and not t.contains(item) {
            finite_set_sum_insert(t, item, nat_one_fn[V])
            finite_set_sum(fs_insert(t, item), nat_one_fn[V]) =
                nat_one_fn[V](item) + finite_set_sum(t, nat_one_fn[V])
            nat_one_fn[V](item) = Nat.1
            finite_set_sum(t, nat_one_fn[V]) = fs_card(t)
            finite_set_sum(fs_insert(t, item), nat_one_fn[V]) = Nat.1 + fs_card(t)
            fs_card_insert_of_not_contains(t, item)
            fs_card(fs_insert(t, item)) = fs_card(t) + Nat.1
            Nat.1 + fs_card(t) = fs_card(t) + Nat.1
            finite_set_sum(fs_insert(t, item), nat_one_fn[V]) = fs_card(fs_insert(t, item))
        }
        p(t) and not t.contains(item) implies p(fs_insert(t, item))
    }
    finite_set_strong_induction(p, s)
    p(s)
    finite_set_sum(s, nat_one_fn[V]) = fs_card(s)
}

/// A sum over a set containing `x` splits off the value at `x`.
theorem finite_set_sum_remove[V](s: FiniteSet[V], x: V, f: V -> Nat) {
    s.contains(x) implies finite_set_sum(s, f) = f(x) + finite_set_sum(fs_remove(s, x), f)
} by {
    if s.contains(x) {
        forall(y: V) {
            fs_insert_contains_eq(fs_remove(s, x), x, y)
            fs_insert(fs_remove(s, x), x).contains(y) =
                (y = x or fs_remove(s, x).contains(y))
            fs_remove_contains_eq(s, x, y)
            fs_remove(s, x).contains(y) = (s.contains(y) and y != x)
            if y = x {
                fs_insert_contains_self(fs_remove(s, x), x)
                fs_insert(fs_remove(s, x), x).contains(x)
                fs_insert(fs_remove(s, x), x).contains(y)
                s.contains(y)
                fs_insert(fs_remove(s, x), x).contains(y) = s.contains(y)
            }
            if y != x {
                fs_insert_contains_eq(fs_remove(s, x), x, y)
                fs_insert(fs_remove(s, x), x).contains(y) =
                    (y = x or fs_remove(s, x).contains(y))
                fs_remove_contains_eq(s, x, y)
                fs_remove(s, x).contains(y) = (s.contains(y) and y != x)
                fs_insert(fs_remove(s, x), x).contains(y) =
                    (y = x or (s.contains(y) and y != x))
                (y = x or (s.contains(y) and y != x)) = s.contains(y)
                fs_insert(fs_remove(s, x), x).contains(y) = s.contains(y)
            }
            fs_insert(fs_remove(s, x), x).contains(y) = s.contains(y)
        }
        finite_set_eq_of_contains_eq(fs_insert(fs_remove(s, x), x), s)
        fs_insert(fs_remove(s, x), x) = s
        finite_set_sum_eq_of_finite_set_eq(fs_insert(fs_remove(s, x), x), s, f)
        finite_set_sum(fs_insert(fs_remove(s, x), x), f) = finite_set_sum(s, f)
        fs_remove_contains_eq(s, x, x)
        fs_remove(s, x).contains(x) = (s.contains(x) and x != x)
        not fs_remove(s, x).contains(x)
        finite_set_sum_insert(fs_remove(s, x), x, f)
        finite_set_sum(fs_insert(fs_remove(s, x), x), f) =
            f(x) + finite_set_sum(fs_remove(s, x), f)
        finite_set_sum(s, f) = f(x) + finite_set_sum(fs_remove(s, x), f)
    }
}

/// The vertices of a tail walk lie in `s` when the vertices of the extended walk do.
theorem walk_vertices_in_set_tail[V](s: FiniteSet[V], prev: V, next: V, rest: List[V]) {
    walk_vertices_in_set(s, prev, List.cons(next, rest)) implies walk_vertices_in_set(s, next, rest)
} by {
    if walk_vertices_in_set(s, prev, List.cons(next, rest)) {
        walk_vertices_in_set(s, prev, List.cons(next, rest)) = forall(x: V) {
            simple_graph_walk_vertices(prev, List.cons(next, rest)).contains(x) implies s.contains(x)
        }
        simple_graph_walk_vertices(prev, List.cons(next, rest)) =
            List.cons(prev, List.cons(next, rest))
        forall(x: V) {
            List.cons(prev, List.cons(next, rest)).contains(x) implies s.contains(x)
        }
        forall(x: V) {
            if simple_graph_walk_vertices(next, rest).contains(x) {
                simple_graph_walk_vertices(next, rest) = List.cons(next, rest)
                List.cons(next, rest).contains(x)
                cons_contains_of_tail_contains(prev, List.cons(next, rest), x)
                List.cons(prev, List.cons(next, rest)).contains(x)
                List.cons(prev, List.cons(next, rest)).contains(x) implies s.contains(x)
                s.contains(x)
            }
            simple_graph_walk_vertices(next, rest).contains(x) implies s.contains(x)
        }
        walk_vertices_in_set(s, next, rest)
    }
}

// Commented out: the proof search cannot handle the nested forall-implies goals in the
// induction step (walk_edges_cons_p_step: "shallow explosion"). The claim is derivable
// from the walk and the vertices-in-set condition; the definition of
// `is_eulerian_circuit` below makes it explicit instead, so nothing downstream needs it.
// /// The induction step for `walk_edges_to_are_directed_edges` at a single pair of
// /// endpoints: a walk extended by one edge still traverses only edges of `(g, s)`.
// theorem walk_edges_cons_directed_step[V](
//     g: SimpleGraph[V], s: FiniteSet[V], a: V, b: V, next: V, tail: List[V]
// ) {
//     (simple_graph_walk(g, next, b, tail) and walk_vertices_in_set(s, next, tail) and
//         s.contains(b) implies
//             forall(q: Pair[V, V]) {
//                 walk_edges(next, tail).contains(q) implies directed_edges(g, s).contains(q)
//             }) and
//     simple_graph_walk(g, a, b, List.cons(next, tail)) and
//     walk_vertices_in_set(s, a, List.cons(next, tail)) and s.contains(b) implies
//         forall(e: Pair[V, V]) {
//             walk_edges(a, List.cons(next, tail)).contains(e) implies directed_edges(g, s).contains(e)
//         }
// } by {
//     if (simple_graph_walk(g, next, b, tail) and walk_vertices_in_set(s, next, tail) and
//             s.contains(b) implies
//                 forall(q: Pair[V, V]) {
//                     walk_edges(next, tail).contains(q) implies directed_edges(g, s).contains(q)
//                 }) and
//         simple_graph_walk(g, a, b, List.cons(next, tail)) and
//         walk_vertices_in_set(s, a, List.cons(next, tail)) and s.contains(b) {
//         simple_graph_walk_cons_iff(g, a, b, next, tail)
//         g.adj(a, next)
//         simple_graph_walk(g, next, b, tail)
//         walk_vertices_in_set(s, a, List.cons(next, tail)) = forall(x: V) {
//             simple_graph_walk_vertices(a, List.cons(next, tail)).contains(x) implies s.contains(x)
//         }
//         simple_graph_walk_vertices(a, List.cons(next, tail)) =
//             List.cons(a, List.cons(next, tail))
//         forall(x: V) {
//             List.cons(a, List.cons(next, tail)).contains(x) implies s.contains(x)
//         }
//         cons_contains_head(a, List.cons(next, tail))
//         List.cons(a, List.cons(next, tail)).contains(a)
//         s.contains(a)
//         cons_contains_eq(a, List.cons(next, tail), next)
//         List.cons(a, List.cons(next, tail)).contains(next) =
//             (a = next or List.cons(next, tail).contains(next))
//         cons_contains_head(next, tail)
//         List.cons(next, tail).contains(next)
//         List.cons(a, List.cons(next, tail)).contains(next)
//         s.contains(next)
//         walk_vertices_in_set_tail(s, a, next, tail)
//         walk_vertices_in_set(s, next, tail)
//         walk_edges_cons(a, next, tail)
//         walk_edges(a, List.cons(next, tail)) =
//             List.cons(Pair.new(a, next), walk_edges(next, tail))
//         forall(e: Pair[V, V]) {
//             if walk_edges(a, List.cons(next, tail)).contains(e) {
//                 walk_edges_cons(a, next, tail)
//                 cons_contains_cases(Pair.new(a, next), walk_edges(next, tail), e)
//                 e = Pair.new(a, next) or walk_edges(next, tail).contains(e)
//                 if e = Pair.new(a, next) {
//                     directed_edges_contains_pair(g, s, a, next)
//                     directed_edges(g, s).contains(Pair.new(a, next))
//                     directed_edges(g, s).contains(e)
//                 }
//                 if e != Pair.new(a, next) {
//                     walk_edges(next, tail).contains(e)
//                     simple_graph_walk(g, next, b, tail) and walk_vertices_in_set(s, next, tail) and s.contains(b) implies forall(q: Pair[V, V]) { walk_edges(next, tail).contains(q) implies directed_edges(g, s).contains(q) }
//                     forall(q: Pair[V, V]) { walk_edges(next, tail).contains(q) implies directed_edges(g, s).contains(q) }
//                     walk_edges(next, tail).contains(e) implies directed_edges(g, s).contains(e)
//                     directed_edges(g, s).contains(e)
//                 }
//                 directed_edges(g, s).contains(e)
//             }
//             walk_edges(a, List.cons(next, tail)).contains(e) implies directed_edges(g, s).contains(e)
//         }
//     }
// }

// /// The induction step for `walk_edges_to_are_directed_edges`, stated at the level of
// /// the induction predicate: the tail property transfers to the extended walk.
// theorem walk_edges_cons_p_step[V](
//     g: SimpleGraph[V], s: FiniteSet[V], next: V, tail: List[V]
// ) {
//     (forall(a0: V, b0: V) {
//         simple_graph_walk(g, a0, b0, tail) and walk_vertices_in_set(s, a0, tail) and
//         s.contains(b0) implies
//             forall(q: Pair[V, V]) {
//                 walk_edges(a0, tail).contains(q) implies directed_edges(g, s).contains(q)
//             }
//     }) implies
//         forall(a0: V, b0: V) {
//             simple_graph_walk(g, a0, b0, List.cons(next, tail)) and
//             walk_vertices_in_set(s, a0, List.cons(next, tail)) and s.contains(b0) implies
//                 forall(e: Pair[V, V]) {
//                     walk_edges(a0, List.cons(next, tail)).contains(e) implies
//                         directed_edges(g, s).contains(e)
//                 }
//         }
// } by {
//     if forall(a0: V, b0: V) {
//         simple_graph_walk(g, a0, b0, tail) and walk_vertices_in_set(s, a0, tail) and
//         s.contains(b0) implies
//             forall(q: Pair[V, V]) {
//                 walk_edges(a0, tail).contains(q) implies directed_edges(g, s).contains(q)
//             }
//     } {
//         forall(a0: V, b0: V) {
//             walk_edges_cons_directed_step(g, s, a0, b0, next, tail)
//             ((simple_graph_walk(g, next, b0, tail) and walk_vertices_in_set(s, next, tail) and s.contains(b0)) implies forall(q: Pair[V, V]) { walk_edges(next, tail).contains(q) implies directed_edges(g, s).contains(q) }) and simple_graph_walk(g, a0, b0, List.cons(next, tail)) and walk_vertices_in_set(s, a0, List.cons(next, tail)) and s.contains(b0) implies forall(e: Pair[V, V]) { walk_edges(a0, List.cons(next, tail)).contains(e) implies directed_edges(g, s).contains(e) }
//         }
//         forall(a0: V, b0: V) {
//             if simple_graph_walk(g, a0, b0, List.cons(next, tail)) and walk_vertices_in_set(s, a0, List.cons(next, tail)) and s.contains(b0) {
//                 simple_graph_walk_cons_iff(g, a0, b0, next, tail)
//                 simple_graph_walk(g, next, b0, tail)
//                 walk_vertices_in_set_tail(s, a0, next, tail)
//                 walk_vertices_in_set(s, next, tail)
//                 forall(a1: V, b1: V) { simple_graph_walk(g, a1, b1, tail) and walk_vertices_in_set(s, a1, tail) and s.contains(b1) implies forall(q: Pair[V, V]) { walk_edges(a1, tail).contains(q) implies directed_edges(g, s).contains(q) } }
//                 (simple_graph_walk(g, next, b0, tail) and walk_vertices_in_set(s, next, tail) and s.contains(b0)) implies forall(q: Pair[V, V]) { walk_edges(next, tail).contains(q) implies directed_edges(g, s).contains(q) }
//                 walk_edges_cons_directed_step(g, s, a0, b0, next, tail)
//                 ((simple_graph_walk(g, next, b0, tail) and walk_vertices_in_set(s, next, tail) and s.contains(b0)) implies forall(q: Pair[V, V]) { walk_edges(next, tail).contains(q) implies directed_edges(g, s).contains(q) }) and simple_graph_walk(g, a0, b0, List.cons(next, tail)) and walk_vertices_in_set(s, a0, List.cons(next, tail)) and s.contains(b0) implies forall(e: Pair[V, V]) { walk_edges(a0, List.cons(next, tail)).contains(e) implies directed_edges(g, s).contains(e) }
//                 ((simple_graph_walk(g, next, b0, tail) and walk_vertices_in_set(s, next, tail) and s.contains(b0)) implies forall(q: Pair[V, V]) { walk_edges(next, tail).contains(q) implies directed_edges(g, s).contains(q) }) and simple_graph_walk(g, a0, b0, List.cons(next, tail)) and walk_vertices_in_set(s, a0, List.cons(next, tail)) and s.contains(b0)
//                 forall(e: Pair[V, V]) { walk_edges(a0, List.cons(next, tail)).contains(e) implies directed_edges(g, s).contains(e) }
//             }
//             simple_graph_walk(g, a0, b0, List.cons(next, tail)) and walk_vertices_in_set(s, a0, List.cons(next, tail)) and s.contains(b0) implies forall(e: Pair[V, V]) { walk_edges(a0, List.cons(next, tail)).contains(e) implies directed_edges(g, s).contains(e) }
//         }
//     }
// }

// /// Every edge traversed by a walk whose vertices lie in `s` is an edge of `(g, s)`.
// theorem walk_edges_to_are_directed_edges[V](
//     g: SimpleGraph[V], s: FiniteSet[V], prev: V, close: V, steps: List[V]
// ) {
//     simple_graph_walk(g, prev, close, steps) and walk_vertices_in_set(s, prev, steps) and
//     s.contains(close) implies
//         forall(e: Pair[V, V]) {
//             walk_edges(prev, steps).contains(e) implies directed_edges(g, s).contains(e)
//         }
// } by {
//     define p(xs: List[V]) -> Bool {
//         forall(a: V, b: V) {
//             simple_graph_walk(g, a, b, xs) and walk_vertices_in_set(s, a, xs) and s.contains(b) implies
//                 forall(e: Pair[V, V]) {
//                     walk_edges(a, xs).contains(e) implies directed_edges(g, s).contains(e)
//                 }
//         }
//     }

//     forall(a: V, b: V) {
//         if simple_graph_walk(g, a, b, List.nil[V]) and walk_vertices_in_set(s, a, List.nil[V]) and
//             s.contains(b) {
//             forall(e: Pair[V, V]) {
//                 walk_edges_nil(a)
//                 walk_edges(a, List.nil[V]) = List.nil[Pair[V, V]]
//                 nil_not_contains(e)
//                 not List.nil[Pair[V, V]].contains(e)
//                 not (List.nil[Pair[V, V]].contains(e) and not directed_edges(g, s).contains(e))
//                 not (walk_edges(a, List.nil[V]).contains(e) and not directed_edges(g, s).contains(e))
//                 if walk_edges(a, List.nil[V]).contains(e) {
//                     not (walk_edges(a, List.nil[V]).contains(e) and not directed_edges(g, s).contains(e))
//                     if not directed_edges(g, s).contains(e) {
//                         walk_edges(a, List.nil[V]).contains(e) and not directed_edges(g, s).contains(e)
//                         false
//                     }
//                     directed_edges(g, s).contains(e)
//                 }
//                 walk_edges(a, List.nil[V]).contains(e) implies directed_edges(g, s).contains(e)
//             }
//         }
//     }
//     p(List.nil[V]) = forall(a0: V, b0: V) {
//         simple_graph_walk(g, a0, b0, List.nil[V]) and walk_vertices_in_set(s, a0, List.nil[V]) and
//         s.contains(b0) implies
//             forall(e: Pair[V, V]) {
//                 walk_edges(a0, List.nil[V]).contains(e) implies directed_edges(g, s).contains(e)
//             }
//     }
//     p(List.nil[V])

//     forall(next: V, tail: List[V]) {
//         if p(tail) {
//             p(tail) = forall(a0: V, b0: V) {
//                 simple_graph_walk(g, a0, b0, tail) and walk_vertices_in_set(s, a0, tail) and
//                 s.contains(b0) implies
//                     forall(q: Pair[V, V]) {
//                         walk_edges(a0, tail).contains(q) implies directed_edges(g, s).contains(q)
//                     }
//             }
//             walk_edges_cons_p_step(g, s, next, tail)
//             forall(a0: V, b0: V) {
//                 simple_graph_walk(g, a0, b0, List.cons(next, tail)) and walk_vertices_in_set(s, a0, List.cons(next, tail)) and s.contains(b0) implies forall(e: Pair[V, V]) { walk_edges(a0, List.cons(next, tail)).contains(e) implies directed_edges(g, s).contains(e) }
//             }
//             p(List.cons(next, tail)) = forall(a0: V, b0: V) {
//                 simple_graph_walk(g, a0, b0, List.cons(next, tail)) and
//                 walk_vertices_in_set(s, a0, List.cons(next, tail)) and s.contains(b0) implies
//                     forall(e: Pair[V, V]) {
//                         walk_edges(a0, List.cons(next, tail)).contains(e) implies
//                             directed_edges(g, s).contains(e)
//                     }
//             }
//             p(List.cons(next, tail))
//         }
//         p(tail) implies p(List.cons(next, tail))
//     }
//     forall(next: V, tail: List[V]) { p(tail) implies p(List.cons(next, tail)) }
//     list_induction_elim(p, steps)
//     p(steps)
//     p(steps) = forall(a: V, b: V) {
//         simple_graph_walk(g, a, b, steps) and walk_vertices_in_set(s, a, steps) and s.contains(b) implies
//             forall(e: Pair[V, V]) {
//                 walk_edges(a, steps).contains(e) implies directed_edges(g, s).contains(e)
//             }
//     }
//     simple_graph_walk(g, prev, close, steps) and walk_vertices_in_set(s, prev, steps) and
//         s.contains(close) implies forall(e: Pair[V, V]) { walk_edges(prev, steps).contains(e) implies directed_edges(g, s).contains(e) }
//     if simple_graph_walk(g, prev, close, steps) and walk_vertices_in_set(s, prev, steps) and
//         s.contains(close) {
//         forall(e: Pair[V, V]) { walk_edges(prev, steps).contains(e) implies directed_edges(g, s).contains(e) }
//     }
// }

// /// Every edge traversed by a closed walk inside `s` is an edge of `(g, s)`.
// theorem walk_edges_are_directed_edges[V](
//     g: SimpleGraph[V], s: FiniteSet[V], start: V, steps: List[V]
// ) {
//     simple_graph_closed_walk(g, start, steps) and walk_vertices_in_set(s, start, steps) implies
//         forall(p: Pair[V, V]) {
//             walk_edges(start, steps).contains(p) implies directed_edges(g, s).contains(p)
//         }
// } by {
//     if simple_graph_closed_walk(g, start, steps) and walk_vertices_in_set(s, start, steps) {
//         simple_graph_closed_walk_is_walk(g, start, steps)
//         simple_graph_walk(g, start, start, steps)
//         walk_vertices_in_set(s, start, steps) = forall(x: V) {
//             simple_graph_walk_vertices(start, steps).contains(x) implies s.contains(x)
//         }
//         simple_graph_walk_vertices(start, steps) = List.cons(start, steps)
//         cons_contains_head(start, steps)
//         List.cons(start, steps).contains(start)
//         s.contains(start)
//         walk_edges_to_are_directed_edges(g, s, start, start, steps)
//         forall(p: Pair[V, V]) {
//             walk_edges(start, steps).contains(p) implies directed_edges(g, s).contains(p)
//         }
//     }
// }

/// Counting a pair inside a prepended edge list: the head contributes exactly when it
/// is the pair being counted.
theorem count_cons_pair_eq[V](head: Pair[V, V], tail: List[Pair[V, V]], v: V, w: V) {
    List.cons(head, tail).count(Pair.new(v, w)) =
        (if head.first = v and head.second = w { Nat.1 } else { Nat.0 }) +
        tail.count(Pair.new(v, w))
} by {
    count_cons_eq_if(head, tail, Pair.new(v, w))
    List.cons(head, tail).count(Pair.new(v, w)) =
        if head = Pair.new(v, w) { Nat.1 + tail.count(Pair.new(v, w)) } else { tail.count(Pair.new(v, w)) }
    if head = Pair.new(v, w) {
        pair_new_first(v, w)
        Pair.new(v, w).first = v
        pair_new_second(v, w)
        Pair.new(v, w).second = w
        head.first = Pair.new(v, w).first
        head.first = v
        head.second = Pair.new(v, w).second
        head.second = w
        (if head.first = v and head.second = w { Nat.1 } else { Nat.0 }) = Nat.1
        (if head.first = v and head.second = w { Nat.1 } else { Nat.0 }) + tail.count(Pair.new(v, w)) =
            Nat.1 + tail.count(Pair.new(v, w))
        List.cons(head, tail).count(Pair.new(v, w)) =
            (if head.first = v and head.second = w { Nat.1 } else { Nat.0 }) +
            tail.count(Pair.new(v, w))
    }
    if head != Pair.new(v, w) {
        if head.first = v and head.second = w {
            pair_eta(head)
            Pair.new(head.first, head.second) = head
            Pair.new(v, w) = head
            false
        }
        not (head.first = v and head.second = w)
        (if head.first = v and head.second = w { Nat.1 } else { Nat.0 }) = Nat.0
        List.cons(head, tail).count(Pair.new(v, w)) = tail.count(Pair.new(v, w))
        List.cons(head, tail).count(Pair.new(v, w)) =
            (if head.first = v and head.second = w { Nat.1 } else { Nat.0 }) +
            tail.count(Pair.new(v, w))
    }
}

/// The traversal count of a prepended edge list splits off the head's contribution.
theorem traversal_count_fn_cons[V](head: Pair[V, V], tail: List[Pair[V, V]], v: V, w: V) {
    traversal_count_fn(List.cons(head, tail), v)(w) =
        (if head.first = v and head.second = w { Nat.1 } else { Nat.0 }) +
        (if head.first = w and head.second = v { Nat.1 } else { Nat.0 }) +
        traversal_count_fn(tail, v)(w)
} by {
    traversal_count_fn(List.cons(head, tail), v)(w) =
        List.cons(head, tail).count(Pair.new(v, w)) + List.cons(head, tail).count(Pair.new(w, v))
    count_cons_pair_eq(head, tail, v, w)
    List.cons(head, tail).count(Pair.new(v, w)) =
        (if head.first = v and head.second = w { Nat.1 } else { Nat.0 }) +
        tail.count(Pair.new(v, w))
    count_cons_pair_eq(head, tail, w, v)
    List.cons(head, tail).count(Pair.new(w, v)) =
        (if head.first = w and head.second = v { Nat.1 } else { Nat.0 }) +
        tail.count(Pair.new(w, v))
    traversal_count_fn(tail, v)(w) = tail.count(Pair.new(v, w)) + tail.count(Pair.new(w, v))
    traversal_count_fn(List.cons(head, tail), v)(w) =
        (if head.first = v and head.second = w { Nat.1 } else { Nat.0 }) +
        tail.count(Pair.new(v, w)) +
        (if head.first = w and head.second = v { Nat.1 } else { Nat.0 }) +
        tail.count(Pair.new(w, v))
    traversal_count_fn(List.cons(head, tail), v)(w) =
        (if head.first = v and head.second = w { Nat.1 } else { Nat.0 }) +
        (if head.first = w and head.second = v { Nat.1 } else { Nat.0 }) +
        traversal_count_fn(tail, v)(w)
}

/// The number of edges of `es` incident to `v` is the sum, over the neighbors of `v`,
/// of how often each incident edge is traversed.
///
/// This is the double count that underlies the forward direction of Euler's theorem:
/// every traversed edge incident to `v` is the undirected edge `{v, w}` for exactly one
/// neighbor `w`, so counting positions and summing traversal counts agree.
theorem walk_edges_incident_count_eq_sum[V](
    g: SimpleGraph[V], s: FiniteSet[V], v: V, es: List[Pair[V, V]]
) {
    (forall(p: Pair[V, V]) { es.contains(p) implies directed_edges(g, s).contains(p) }) implies
        es.filter(incident_edge_pred(v)).length =
        finite_set_sum(neighborhood(g, s, v), traversal_count_fn(es, v))
} by {
    define pc(xs: List[Pair[V, V]]) -> Bool {
        (forall(p: Pair[V, V]) { xs.contains(p) implies directed_edges(g, s).contains(p) }) implies
            xs.filter(incident_edge_pred(v)).length =
            finite_set_sum(neighborhood(g, s, v), traversal_count_fn(xs, v))
    }

    forall(w: V) {
        if neighborhood(g, s, v).contains(w) {
            traversal_count_fn(List.nil[Pair[V, V]], v)(w) =
                List.nil[Pair[V, V]].count(Pair.new(v, w)) +
                List.nil[Pair[V, V]].count(Pair.new(w, v))
            List.nil[Pair[V, V]].count(Pair.new(v, w)) = Nat.0
            List.nil[Pair[V, V]].count(Pair.new(w, v)) = Nat.0
            traversal_count_fn(List.nil[Pair[V, V]], v)(w) = Nat.0
            nat_zero_fn[V](w) = Nat.0
            traversal_count_fn(List.nil[Pair[V, V]], v)(w) = nat_zero_fn[V](w)
        }
        neighborhood(g, s, v).contains(w) implies traversal_count_fn(List.nil[Pair[V, V]], v)(w) = nat_zero_fn[V](w)
    }
    finite_set_sum_pointwise_eq(neighborhood(g, s, v),
        traversal_count_fn(List.nil[Pair[V, V]], v), nat_zero_fn[V])
    finite_set_sum(neighborhood(g, s, v), traversal_count_fn(List.nil[Pair[V, V]], v)) =
        finite_set_sum(neighborhood(g, s, v), nat_zero_fn[V])
    finite_set_sum_zero(neighborhood(g, s, v))
    finite_set_sum(neighborhood(g, s, v), nat_zero_fn[V]) = Nat.0
    List.nil[Pair[V, V]].filter(incident_edge_pred(v)) = List.nil[Pair[V, V]]
    List.nil[Pair[V, V]].filter(incident_edge_pred(v)).length = Nat.0
    List.nil[Pair[V, V]].filter(incident_edge_pred(v)).length =
        finite_set_sum(neighborhood(g, s, v), traversal_count_fn(List.nil[Pair[V, V]], v))
    pc(List.nil[Pair[V, V]])

    forall(head: Pair[V, V], tail: List[Pair[V, V]]) {
        if pc(tail) {
            if forall(p: Pair[V, V]) {
                List.cons(head, tail).contains(p) implies directed_edges(g, s).contains(p)
            } {
                cons_contains_head(head, tail)
                List.cons(head, tail).contains(head)
                directed_edges(g, s).contains(head)
                forall(p: Pair[V, V]) {
                    if tail.contains(p) {
                        cons_contains_of_tail_contains(head, tail, p)
                        List.cons(head, tail).contains(p)
                        List.cons(head, tail).contains(p) implies directed_edges(g, s).contains(p)
                        directed_edges(g, s).contains(p)
                    }
                    tail.contains(p) implies directed_edges(g, s).contains(p)
                }
                pc(tail)
                pc(tail) = ((forall(p: Pair[V, V]) {
                    tail.contains(p) implies directed_edges(g, s).contains(p)
                }) implies tail.filter(incident_edge_pred(v)).length =
                    finite_set_sum(neighborhood(g, s, v), traversal_count_fn(tail, v)))
                tail.filter(incident_edge_pred(v)).length =
                    finite_set_sum(neighborhood(g, s, v), traversal_count_fn(tail, v))
                directed_edges_ne(g, s, head)
                head.first != head.second

                if head.first = v {
                    let w0: V = head.second
                    pair_eta(head)
                    Pair.new(head.first, head.second) = head
                    Pair.new(v, w0) = head
                    directed_edges_contains_eq(g, s, head)
                    directed_edges(g, s).contains(head) =
                        (s.contains(head.first) and s.contains(head.second) and
                            g.adj(head.first, head.second))
                    s.contains(w0)
                    g.adj(v, w0)
                    neighborhood_contains_eq(g, s, v, w0)
                    neighborhood(g, s, v).contains(w0) = (s.contains(w0) and g.adj(v, w0))
                    neighborhood(g, s, v).contains(w0)
                    w0 != v
                    incident_edge_pred(v)(head) = (head.first = v or head.second = v)
                    incident_edge_pred(v)(head)
                    filter_cons_of_true(head, tail, incident_edge_pred(v))
                    List.cons(head, tail).filter(incident_edge_pred(v)) =
                        List.cons(head, tail.filter(incident_edge_pred(v)))
                    List.cons(head, tail).filter(incident_edge_pred(v)).length =
                        tail.filter(incident_edge_pred(v)).length.suc
                    List.cons(head, tail).filter(incident_edge_pred(v)).length =
                        tail.filter(incident_edge_pred(v)).length + Nat.1
                    List.cons(head, tail).filter(incident_edge_pred(v)).length =
                        Nat.1 + finite_set_sum(neighborhood(g, s, v), traversal_count_fn(tail, v))
                    traversal_count_fn_cons(head, tail, v, w0)
                    traversal_count_fn(List.cons(head, tail), v)(w0) =
                        (if head.first = v and head.second = w0 { Nat.1 } else { Nat.0 }) +
                        (if head.first = w0 and head.second = v { Nat.1 } else { Nat.0 }) +
                        traversal_count_fn(tail, v)(w0)
                    (if head.first = v and head.second = w0 { Nat.1 } else { Nat.0 }) = Nat.1
                    (if head.first = w0 and head.second = v { Nat.1 } else { Nat.0 }) = Nat.0
                    traversal_count_fn(List.cons(head, tail), v)(w0) =
                        Nat.1 + traversal_count_fn(tail, v)(w0)
                    forall(w: V) {
                        if fs_remove(neighborhood(g, s, v), w0).contains(w) {
                            fs_remove_contains_eq(neighborhood(g, s, v), w0, w)
                            fs_remove(neighborhood(g, s, v), w0).contains(w) =
                                (neighborhood(g, s, v).contains(w) and w != w0)
                            w != w0
                            traversal_count_fn_cons(head, tail, v, w)
                            traversal_count_fn(List.cons(head, tail), v)(w) =
                                (if head.first = v and head.second = w { Nat.1 } else { Nat.0 }) +
                                (if head.first = w and head.second = v { Nat.1 } else { Nat.0 }) +
                                traversal_count_fn(tail, v)(w)
                            (if head.first = v and head.second = w { Nat.1 } else { Nat.0 }) = Nat.0
                            (if head.first = w and head.second = v { Nat.1 } else { Nat.0 }) = Nat.0
                            traversal_count_fn(List.cons(head, tail), v)(w) =
                                traversal_count_fn(tail, v)(w)
                        }
                        fs_remove(neighborhood(g, s, v), w0).contains(w) implies traversal_count_fn(List.cons(head, tail), v)(w) = traversal_count_fn(tail, v)(w)
                    }
                    finite_set_sum_pointwise_eq(fs_remove(neighborhood(g, s, v), w0),
                        traversal_count_fn(List.cons(head, tail), v), traversal_count_fn(tail, v))
                    finite_set_sum(fs_remove(neighborhood(g, s, v), w0),
                        traversal_count_fn(List.cons(head, tail), v)) =
                        finite_set_sum(fs_remove(neighborhood(g, s, v), w0),
                            traversal_count_fn(tail, v))
                    finite_set_sum_remove(neighborhood(g, s, v), w0,
                        traversal_count_fn(List.cons(head, tail), v))
                    finite_set_sum(neighborhood(g, s, v),
                        traversal_count_fn(List.cons(head, tail), v)) =
                        traversal_count_fn(List.cons(head, tail), v)(w0) +
                        finite_set_sum(fs_remove(neighborhood(g, s, v), w0),
                            traversal_count_fn(List.cons(head, tail), v))
                    finite_set_sum(neighborhood(g, s, v),
                        traversal_count_fn(List.cons(head, tail), v)) =
                        Nat.1 + traversal_count_fn(tail, v)(w0) +
                        finite_set_sum(fs_remove(neighborhood(g, s, v), w0),
                            traversal_count_fn(tail, v))
                    finite_set_sum_remove(neighborhood(g, s, v), w0, traversal_count_fn(tail, v))
                    finite_set_sum(neighborhood(g, s, v), traversal_count_fn(tail, v)) =
                        traversal_count_fn(tail, v)(w0) +
                        finite_set_sum(fs_remove(neighborhood(g, s, v), w0),
                            traversal_count_fn(tail, v))
                    finite_set_sum(neighborhood(g, s, v),
                        traversal_count_fn(List.cons(head, tail), v)) =
                        Nat.1 + finite_set_sum(neighborhood(g, s, v), traversal_count_fn(tail, v))
                    List.cons(head, tail).filter(incident_edge_pred(v)).length =
                        finite_set_sum(neighborhood(g, s, v),
                            traversal_count_fn(List.cons(head, tail), v))
                }
                if head.first != v and head.second = v {
                    let w0: V = head.first
                    pair_eta(head)
                    Pair.new(head.first, head.second) = head
                    Pair.new(w0, v) = head
                    directed_edges_contains_eq(g, s, head)
                    directed_edges(g, s).contains(head) =
                        (s.contains(head.first) and s.contains(head.second) and
                            g.adj(head.first, head.second))
                    s.contains(w0)
                    g.adj(w0, v)
                    neighborhood_contains_eq(g, s, v, w0)
                    neighborhood(g, s, v).contains(w0) = (s.contains(w0) and g.adj(v, w0))
                    simple_graph_adj_comm(g, w0, v)
                    g.adj(v, w0) = g.adj(w0, v)
                    neighborhood(g, s, v).contains(w0)
                    w0 != v
                    incident_edge_pred(v)(head) = (head.first = v or head.second = v)
                    incident_edge_pred(v)(head)
                    filter_cons_of_true(head, tail, incident_edge_pred(v))
                    List.cons(head, tail).filter(incident_edge_pred(v)) =
                        List.cons(head, tail.filter(incident_edge_pred(v)))
                    List.cons(head, tail).filter(incident_edge_pred(v)).length =
                        tail.filter(incident_edge_pred(v)).length.suc
                    List.cons(head, tail).filter(incident_edge_pred(v)).length =
                        tail.filter(incident_edge_pred(v)).length + Nat.1
                    List.cons(head, tail).filter(incident_edge_pred(v)).length =
                        Nat.1 + finite_set_sum(neighborhood(g, s, v), traversal_count_fn(tail, v))
                    traversal_count_fn_cons(head, tail, v, w0)
                    traversal_count_fn(List.cons(head, tail), v)(w0) =
                        (if head.first = v and head.second = w0 { Nat.1 } else { Nat.0 }) +
                        (if head.first = w0 and head.second = v { Nat.1 } else { Nat.0 }) +
                        traversal_count_fn(tail, v)(w0)
                    (if head.first = v and head.second = w0 { Nat.1 } else { Nat.0 }) = Nat.0
                    (if head.first = w0 and head.second = v { Nat.1 } else { Nat.0 }) = Nat.1
                    traversal_count_fn(List.cons(head, tail), v)(w0) =
                        Nat.1 + traversal_count_fn(tail, v)(w0)
                    forall(w: V) {
                        if fs_remove(neighborhood(g, s, v), w0).contains(w) {
                            fs_remove_contains_eq(neighborhood(g, s, v), w0, w)
                            fs_remove(neighborhood(g, s, v), w0).contains(w) =
                                (neighborhood(g, s, v).contains(w) and w != w0)
                            w != w0
                            traversal_count_fn_cons(head, tail, v, w)
                            traversal_count_fn(List.cons(head, tail), v)(w) =
                                (if head.first = v and head.second = w { Nat.1 } else { Nat.0 }) +
                                (if head.first = w and head.second = v { Nat.1 } else { Nat.0 }) +
                                traversal_count_fn(tail, v)(w)
                            (if head.first = v and head.second = w { Nat.1 } else { Nat.0 }) = Nat.0
                            (if head.first = w and head.second = v { Nat.1 } else { Nat.0 }) = Nat.0
                            traversal_count_fn(List.cons(head, tail), v)(w) =
                                traversal_count_fn(tail, v)(w)
                        }
                        fs_remove(neighborhood(g, s, v), w0).contains(w) implies traversal_count_fn(List.cons(head, tail), v)(w) = traversal_count_fn(tail, v)(w)
                    }
                    finite_set_sum_pointwise_eq(fs_remove(neighborhood(g, s, v), w0),
                        traversal_count_fn(List.cons(head, tail), v), traversal_count_fn(tail, v))
                    finite_set_sum(fs_remove(neighborhood(g, s, v), w0),
                        traversal_count_fn(List.cons(head, tail), v)) =
                        finite_set_sum(fs_remove(neighborhood(g, s, v), w0),
                            traversal_count_fn(tail, v))
                    finite_set_sum_remove(neighborhood(g, s, v), w0,
                        traversal_count_fn(List.cons(head, tail), v))
                    finite_set_sum(neighborhood(g, s, v),
                        traversal_count_fn(List.cons(head, tail), v)) =
                        traversal_count_fn(List.cons(head, tail), v)(w0) +
                        finite_set_sum(fs_remove(neighborhood(g, s, v), w0),
                            traversal_count_fn(List.cons(head, tail), v))
                    finite_set_sum(neighborhood(g, s, v),
                        traversal_count_fn(List.cons(head, tail), v)) =
                        Nat.1 + traversal_count_fn(tail, v)(w0) +
                        finite_set_sum(fs_remove(neighborhood(g, s, v), w0),
                            traversal_count_fn(tail, v))
                    finite_set_sum_remove(neighborhood(g, s, v), w0, traversal_count_fn(tail, v))
                    finite_set_sum(neighborhood(g, s, v), traversal_count_fn(tail, v)) =
                        traversal_count_fn(tail, v)(w0) +
                        finite_set_sum(fs_remove(neighborhood(g, s, v), w0),
                            traversal_count_fn(tail, v))
                    finite_set_sum(neighborhood(g, s, v),
                        traversal_count_fn(List.cons(head, tail), v)) =
                        Nat.1 + finite_set_sum(neighborhood(g, s, v), traversal_count_fn(tail, v))
                    List.cons(head, tail).filter(incident_edge_pred(v)).length =
                        finite_set_sum(neighborhood(g, s, v),
                            traversal_count_fn(List.cons(head, tail), v))
                }
                if head.first != v and head.second != v {
                    not incident_edge_pred(v)(head)
                    filter_cons_of_false(head, tail, incident_edge_pred(v))
                    List.cons(head, tail).filter(incident_edge_pred(v)) =
                        tail.filter(incident_edge_pred(v))
                    List.cons(head, tail).filter(incident_edge_pred(v)).length =
                        tail.filter(incident_edge_pred(v)).length
                    List.cons(head, tail).filter(incident_edge_pred(v)).length =
                        finite_set_sum(neighborhood(g, s, v), traversal_count_fn(tail, v))
                    forall(w: V) {
                        if neighborhood(g, s, v).contains(w) {
                            traversal_count_fn_cons(head, tail, v, w)
                            traversal_count_fn(List.cons(head, tail), v)(w) =
                                (if head.first = v and head.second = w { Nat.1 } else { Nat.0 }) +
                                (if head.first = w and head.second = v { Nat.1 } else { Nat.0 }) +
                                traversal_count_fn(tail, v)(w)
                            (if head.first = v and head.second = w { Nat.1 } else { Nat.0 }) = Nat.0
                            (if head.first = w and head.second = v { Nat.1 } else { Nat.0 }) = Nat.0
                            traversal_count_fn(List.cons(head, tail), v)(w) =
                                traversal_count_fn(tail, v)(w)
                        }
                        neighborhood(g, s, v).contains(w) implies traversal_count_fn(List.cons(head, tail), v)(w) = traversal_count_fn(tail, v)(w)
                    }
                    finite_set_sum_pointwise_eq(neighborhood(g, s, v),
                        traversal_count_fn(List.cons(head, tail), v), traversal_count_fn(tail, v))
                    finite_set_sum(neighborhood(g, s, v),
                        traversal_count_fn(List.cons(head, tail), v)) =
                        finite_set_sum(neighborhood(g, s, v), traversal_count_fn(tail, v))
                    List.cons(head, tail).filter(incident_edge_pred(v)).length =
                        finite_set_sum(neighborhood(g, s, v),
                            traversal_count_fn(List.cons(head, tail), v))
                }
                List.cons(head, tail).filter(incident_edge_pred(v)).length =
                    finite_set_sum(neighborhood(g, s, v),
                        traversal_count_fn(List.cons(head, tail), v))
            }
            pc(List.cons(head, tail))
        }
    }
    forall(head: Pair[V, V], tail: List[Pair[V, V]]) {
        pc(tail) implies pc(List.cons(head, tail))
    }
    list_induction_elim(pc, es)
    pc(es)
    if forall(p: Pair[V, V]) { es.contains(p) implies directed_edges(g, s).contains(p) } {
        pc(es) = ((forall(p: Pair[V, V]) {
            es.contains(p) implies directed_edges(g, s).contains(p)
        }) implies es.filter(incident_edge_pred(v)).length =
            finite_set_sum(neighborhood(g, s, v), traversal_count_fn(es, v)))
        es.filter(incident_edge_pred(v)).length =
            finite_set_sum(neighborhood(g, s, v), traversal_count_fn(es, v))
    }
}

/// The forward direction of Euler's theorem, part one: the degree of a vertex equals
/// the number of circuit edges incident to it.
///
/// An Eulerian circuit traverses every edge of `(g, s)` exactly once and stays inside
/// `s`, so the traversed edges incident to `v` are exactly the edges of `(g, s)`
/// incident to `v`. Counting those traversals over the neighbors of `v` gives the
/// degree, by the double count.
theorem eulerian_degree_eq_visit_count[V](
    g: SimpleGraph[V], s: FiniteSet[V], start: V, steps: List[V], v: V
) {
    is_eulerian_circuit(g, s, start, steps) and s.contains(v) implies
        degree(g, s, v) = walk_edge_visit_count(start, steps, v)
} by {
    if is_eulerian_circuit(g, s, start, steps) and s.contains(v) {
        is_eulerian_circuit(g, s, start, steps) = (
            simple_graph_closed_walk(g, start, steps) and
            walk_vertices_in_set(s, start, steps) and
            forall(p: Pair[V, V]) {
                walk_edges(start, steps).contains(p) implies directed_edges(g, s).contains(p)
            } and
            forall(x: V, y: V) {
                directed_edges(g, s).contains(Pair.new(x, y)) implies
                    walk_edge_traversal_count(start, steps, x, y) = Nat.1
            }
        )
        simple_graph_closed_walk(g, start, steps)
        walk_vertices_in_set(s, start, steps)
        forall(p: Pair[V, V]) {
            walk_edges(start, steps).contains(p) implies directed_edges(g, s).contains(p)
        }
        forall(x: V, y: V) {
            directed_edges(g, s).contains(Pair.new(x, y)) implies walk_edge_traversal_count(start, steps, x, y) = Nat.1
        }
        walk_edges_incident_count_eq_sum(g, s, v, walk_edges(start, steps))
        walk_edges(start, steps).filter(incident_edge_pred(v)).length =
            finite_set_sum(neighborhood(g, s, v),
                traversal_count_fn(walk_edges(start, steps), v))
        forall(w: V) {
            if neighborhood(g, s, v).contains(w) {
                neighborhood_contains_eq(g, s, v, w)
                neighborhood(g, s, v).contains(w) = (s.contains(w) and g.adj(v, w))
                s.contains(w)
                g.adj(v, w)
                directed_edges_contains_pair(g, s, v, w)
                directed_edges(g, s).contains(Pair.new(v, w))
                forall(x: V, y: V) {
                    directed_edges(g, s).contains(Pair.new(x, y)) implies walk_edge_traversal_count(start, steps, x, y) = Nat.1
                }
                walk_edge_traversal_count(start, steps, v, w) = Nat.1
                walk_edge_traversal_count(start, steps, v, w) =
                    traversal_count_fn(walk_edges(start, steps), v)(w)
                traversal_count_fn(walk_edges(start, steps), v)(w) = Nat.1
                nat_one_fn[V](w) = Nat.1
                traversal_count_fn(walk_edges(start, steps), v)(w) = nat_one_fn[V](w)
            }
            neighborhood(g, s, v).contains(w) implies traversal_count_fn(walk_edges(start, steps), v)(w) = nat_one_fn[V](w)
        }
        finite_set_sum_pointwise_eq(neighborhood(g, s, v),
            traversal_count_fn(walk_edges(start, steps), v), nat_one_fn[V])
        finite_set_sum(neighborhood(g, s, v),
            traversal_count_fn(walk_edges(start, steps), v)) =
            finite_set_sum(neighborhood(g, s, v), nat_one_fn[V])
        finite_set_sum_one(neighborhood(g, s, v))
        finite_set_sum(neighborhood(g, s, v), nat_one_fn[V]) = fs_card(neighborhood(g, s, v))
        finite_set_sum(neighborhood(g, s, v),
            traversal_count_fn(walk_edges(start, steps), v)) = fs_card(neighborhood(g, s, v))
        degree_eq_neighborhood_card(g, s, v)
        degree(g, s, v) = fs_card(neighborhood(g, s, v))
        walk_edge_visit_count(start, steps, v) =
            walk_edges(start, steps).filter(incident_edge_pred(v)).length
        walk_edge_visit_count(start, steps, v) =
            finite_set_sum(neighborhood(g, s, v),
                traversal_count_fn(walk_edges(start, steps), v))
        walk_edge_visit_count(start, steps, v) = fs_card(neighborhood(g, s, v))
        degree(g, s, v) = walk_edge_visit_count(start, steps, v)
    }
}

/// The forward direction of Euler's theorem: a finite graph with an Eulerian circuit
/// has every vertex of even degree.
///
/// Walking the circuit, each visit to a vertex consumes two incident edges: the edge
/// that arrives and the edge that leaves. The closed-walk identity counts the circuit
/// edges incident to `v` as twice the number of occurrences of `v` among the walk's
/// targets, and the previous theorem identifies that count with the degree.
theorem eulerian_circuit_degree_is_even[V](
    g: SimpleGraph[V], s: FiniteSet[V], start: V, steps: List[V], v: V
) {
    is_eulerian_circuit(g, s, start, steps) and s.contains(v) implies
        exists(k: Nat) { degree(g, s, v) = Nat.2 * k }
} by {
    if is_eulerian_circuit(g, s, start, steps) and s.contains(v) {
        is_eulerian_circuit(g, s, start, steps) = (
            simple_graph_closed_walk(g, start, steps) and
            walk_vertices_in_set(s, start, steps) and
            forall(x: V, y: V) {
                directed_edges(g, s).contains(Pair.new(x, y)) implies
                    walk_edge_traversal_count(start, steps, x, y) = Nat.1
            }
        )
        simple_graph_closed_walk(g, start, steps)
        eulerian_degree_eq_visit_count(g, s, start, steps, v)
        degree(g, s, v) = walk_edge_visit_count(start, steps, v)
        simple_graph_closed_walk_is_walk(g, start, steps)
        simple_graph_walk(g, start, start, steps)
        walk_edge_visit_count_closed(g, start, steps, v)
        walk_edge_visit_count(start, steps, v) = Nat.2 * steps.count(v)
        degree(g, s, v) = Nat.2 * steps.count(v)
        exists(k: Nat) { degree(g, s, v) = Nat.2 * k }
    }
}


// ---------------------------------------------------------------------------
// The converse direction of Euler's theorem and the trail theorem.
//
// These statements are left commented out: the forward direction above is the
// derivable part of Euler's theorem, while the converse requires a constructive
// proof that builds a circuit by extending a trail to maximal length and
// splicing in leftover cycles. The formal content is stated here for reference.
// ---------------------------------------------------------------------------

// /// True of the vertices of `s` whose degree is even.
// define has_even_degree[V](g: SimpleGraph[V], s: FiniteSet[V], v: V) -> Bool {
//     exists(k: Nat) { degree(g, s, v) = Nat.2 * k }
// }
//
// /// Euler's theorem: a finite connected graph has an Eulerian circuit exactly when
// /// every vertex has even degree.
// ///
// /// The forward direction is `eulerian_circuit_degree_is_even`. The converse is the
// /// constructive half: start a trail at any vertex and extend it while an unused
// /// incident edge exists; it must return to its start (an open endpoint would have
// /// odd degree among the used edges), and any leftover edge on a visited vertex can
// /// be spliced into the circuit.
// theorem eulerian_circuit_iff_even_degrees[V](
//     g: SimpleGraph[V], s: FiniteSet[V], start: V
// ) {
//     simple_graph_connected(g) and s.contains(start) implies
//         (exists(steps: List[V]) { is_eulerian_circuit(g, s, start, steps) }) =
//         (forall(v: V) { s.contains(v) implies has_even_degree(g, s, v) })
// }

// /// Euler's trail theorem: a finite connected graph has an Eulerian trail exactly
// /// when it has exactly zero or two vertices of odd degree.
// ///
// /// An Eulerian trail is an open walk that traverses every edge exactly once; it
// /// exists exactly when at most two vertices have odd degree, and if two exist the
// /// trail must start and end at them. Adding an artificial edge between the two odd
// /// vertices reduces this to the circuit theorem above.
// define is_eulerian_trail[V](g: SimpleGraph[V], s: FiniteSet[V], start: V, finish: V, steps: List[V]) -> Bool {
//     simple_graph_walk(g, start, finish, steps) and
//     walk_vertices_in_set(s, start, steps) and
//     forall(p: Pair[V, V]) {
//         walk_edges(start, steps).contains(p) implies directed_edges(g, s).contains(p)
//     } and
//     forall(x: V, y: V) {
//         directed_edges(g, s).contains(Pair.new(x, y)) implies
//             walk_edge_traversal_count(start, steps, x, y) = Nat.1
//     }
// }
