from nat import Nat, lte_antisymm
from finite_set import FiniteSet, finite_set_subset_antisymm, finite_set_subset_contains,
    finite_set_singleton_contains_eq
from data.finite.finite_set_card import fs_card, fs_card_singleton
from data.finite.finite_set_card_compare import fs_card_mono
from data.finite.finite_set_subset_intro import fs_subset_eq_intro
from graph.simple_graph import SimpleGraph, empty_graph, empty_graph_adj_false, complete_graph,
    complete_graph_adj_iff_ne
from graph.simple_graph_domination import is_dominating_set, is_dominating_set_apply,
    is_dominating_set_intro, is_dominated, is_dominated_of_contains, is_dominated_of_adj,
    has_neighbor_in, has_neighbor_in_witness
from graph.simple_graph_domination_number import domination_number, domination_number_is_least,
    domination_number_le_ambient
from graph.simple_graph_minimum_dominating import is_minimum_dominating_set,
    is_minimum_dominating_set_apply, is_dominating_subset, is_dominating_subset_apply,
    minimum_dominating_set_exists

numerals Nat

/// A dominating set of a nonempty ambient set is nonempty.
///
/// A vertex is dominated either by lying in the set or by having a neighbor there, and both
/// readings put something in the set.
theorem dominating_set_contains_some[V](
    g: SimpleGraph[V], s: FiniteSet[V], d: FiniteSet[V], v: V
) {
    is_dominating_set(g, s, d) and s.contains(v) implies exists(w: V) { d.contains(w) }
} by {
    if is_dominating_set(g, s, d) and s.contains(v) {
        is_dominating_set_apply(g, s, d, v)
        is_dominated(g, d, v)
        if d.contains(v) {
            exists(w: V) { d.contains(w) }
        }
        if not d.contains(v) {
            has_neighbor_in(g, d, v)
            has_neighbor_in_witness(g, d, v)
            let (u: V) satisfy {
                d.contains(u) and g.adj(u, v)
            }
            exists(w: V) { d.contains(w) }
        }
        exists(w: V) { d.contains(w) }
    }
}

/// A dominating set of a nonempty ambient set has at least one vertex.
///
/// The singleton of any of its members sits inside it, and counting is monotone.
theorem dominating_set_card_positive[V](
    g: SimpleGraph[V], s: FiniteSet[V], d: FiniteSet[V], v: V
) {
    is_dominating_set(g, s, d) and s.contains(v) implies Nat.1 <= fs_card(d)
} by {
    if is_dominating_set(g, s, d) and s.contains(v) {
        dominating_set_contains_some(g, s, d, v)
        let (w: V) satisfy {
            d.contains(w)
        }
        forall(x: V) {
            if FiniteSet.empty[V].insert(w).contains(x) {
                finite_set_singleton_contains_eq(w, x)
                x = w
                d.contains(x)
            }
            FiniteSet.empty[V].insert(w).contains(x) implies d.contains(x)
        }
        fs_subset_eq_intro(FiniteSet.empty[V].insert(w), d)
        FiniteSet.empty[V].insert(w).subset_eq(d)
        fs_card_mono(FiniteSet.empty[V].insert(w), d)
        fs_card(FiniteSet.empty[V].insert(w)) <= fs_card(d)
        fs_card_singleton(w)
        fs_card(FiniteSet.empty[V].insert(w)) = Nat.1
        Nat.1 <= fs_card(d)
    }
}

/// The domination number of a nonempty vertex set is at least one.
theorem domination_number_positive[V](g: SimpleGraph[V], s: FiniteSet[V], v: V) {
    s.contains(v) implies Nat.1 <= domination_number(g, s)
} by {
    if s.contains(v) {
        minimum_dominating_set_exists(g, s)
        let (d: FiniteSet[V]) satisfy {
            is_minimum_dominating_set(g, s, d)
        }
        is_minimum_dominating_set_apply(g, s, d)
        is_dominating_subset(g, s, d)
        fs_card(d) = domination_number(g, s)
        is_dominating_subset_apply(g, s, d)
        is_dominating_set(g, s, d)
        dominating_set_card_positive(g, s, d, v)
        Nat.1 <= fs_card(d)
        Nat.1 <= domination_number(g, s)
    }
}

/// In the empty graph a vertex is dominated only by lying in the dominating set.
///
/// There are no edges, so the second half of `is_dominated` never applies.
theorem empty_graph_is_dominated_eq[V](d: FiniteSet[V], v: V) {
    is_dominated(empty_graph[V], d, v) = d.contains(v)
} by {
    if is_dominated(empty_graph[V], d, v) {
        if not d.contains(v) {
            has_neighbor_in(empty_graph[V], d, v)
            has_neighbor_in_witness(empty_graph[V], d, v)
            let (w: V) satisfy {
                d.contains(w) and empty_graph[V].adj(w, v)
            }
            empty_graph_adj_false(w, v)
            not empty_graph[V].adj(w, v)
            false
        }
        d.contains(v)
    }
    if d.contains(v) {
        is_dominated_of_contains(empty_graph[V], d, v)
        is_dominated(empty_graph[V], d, v)
    }
    (is_dominated(empty_graph[V], d, v) implies d.contains(v))
    (d.contains(v) implies is_dominated(empty_graph[V], d, v))
    is_dominated(empty_graph[V], d, v) = d.contains(v)
}

/// The only dominating subset of the empty graph is the whole vertex set.
theorem empty_graph_dominating_subset_eq[V](s: FiniteSet[V], d: FiniteSet[V]) {
    is_dominating_subset(empty_graph[V], s, d) implies d = s
} by {
    if is_dominating_subset(empty_graph[V], s, d) {
        is_dominating_subset_apply(empty_graph[V], s, d)
        d.subset_eq(s)
        is_dominating_set(empty_graph[V], s, d)
        forall(v: V) {
            if s.contains(v) {
                is_dominating_set_apply(empty_graph[V], s, d, v)
                is_dominated(empty_graph[V], d, v)
                empty_graph_is_dominated_eq(d, v)
                d.contains(v)
            }
            s.contains(v) implies d.contains(v)
        }
        fs_subset_eq_intro(s, d)
        s.subset_eq(d)
        finite_set_subset_antisymm(d, s)
        d = s
    }
}

/// The domination number of the empty graph is the number of vertices.
///
/// With no edges, no vertex can help dominate another, so every vertex must be chosen.
theorem empty_graph_domination_number[V](s: FiniteSet[V]) {
    domination_number(empty_graph[V], s) = fs_card(s)
} by {
    minimum_dominating_set_exists(empty_graph[V], s)
    let (d: FiniteSet[V]) satisfy {
        is_minimum_dominating_set(empty_graph[V], s, d)
    }
    is_minimum_dominating_set_apply(empty_graph[V], s, d)
    is_dominating_subset(empty_graph[V], s, d)
    fs_card(d) = domination_number(empty_graph[V], s)
    empty_graph_dominating_subset_eq(s, d)
    d = s
    fs_card(s) = domination_number(empty_graph[V], s)
}

/// A single vertex dominates the complete graph.
///
/// Every other vertex is adjacent to it, since adjacency in the complete graph is exactly
/// being distinct.
theorem complete_graph_singleton_dominates[V](s: FiniteSet[V], v: V) {
    is_dominating_set(complete_graph[V], s, FiniteSet.empty[V].insert(v))
} by {
    forall(u: V) {
        if s.contains(u) {
            finite_set_singleton_contains_eq(v, v)
            FiniteSet.empty[V].insert(v).contains(v)
            if u = v {
                is_dominated_of_contains(complete_graph[V], FiniteSet.empty[V].insert(v), u)
                is_dominated(complete_graph[V], FiniteSet.empty[V].insert(v), u)
            }
            if u != v {
                complete_graph_adj_iff_ne(v, u)
                complete_graph[V].adj(v, u) = (v != u)
                complete_graph[V].adj(v, u)
                is_dominated_of_adj(complete_graph[V], FiniteSet.empty[V].insert(v), u, v)
                is_dominated(complete_graph[V], FiniteSet.empty[V].insert(v), u)
            }
            is_dominated(complete_graph[V], FiniteSet.empty[V].insert(v), u)
        }
        s.contains(u) implies is_dominated(complete_graph[V], FiniteSet.empty[V].insert(v), u)
    }
    is_dominating_set_intro(complete_graph[V], s, FiniteSet.empty[V].insert(v))
    is_dominating_set(complete_graph[V], s, FiniteSet.empty[V].insert(v))
}

/// The domination number of a nonempty complete graph is one.
theorem complete_graph_domination_number[V](s: FiniteSet[V], v: V) {
    s.contains(v) implies domination_number(complete_graph[V], s) = Nat.1
} by {
    if s.contains(v) {
        forall(x: V) {
            if FiniteSet.empty[V].insert(v).contains(x) {
                finite_set_singleton_contains_eq(v, x)
                x = v
                s.contains(x)
            }
            FiniteSet.empty[V].insert(v).contains(x) implies s.contains(x)
        }
        fs_subset_eq_intro(FiniteSet.empty[V].insert(v), s)
        FiniteSet.empty[V].insert(v).subset_eq(s)
        complete_graph_singleton_dominates(s, v)
        is_dominating_set(complete_graph[V], s, FiniteSet.empty[V].insert(v))
        domination_number_is_least(complete_graph[V], s, FiniteSet.empty[V].insert(v))
        domination_number(complete_graph[V], s) <= fs_card(FiniteSet.empty[V].insert(v))
        fs_card_singleton(v)
        fs_card(FiniteSet.empty[V].insert(v)) = Nat.1
        domination_number(complete_graph[V], s) <= Nat.1
        domination_number_positive(complete_graph[V], s, v)
        Nat.1 <= domination_number(complete_graph[V], s)
        lte_antisymm(domination_number(complete_graph[V], s), Nat.1)
        domination_number(complete_graph[V], s) = Nat.1
    }
}
