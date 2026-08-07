from nat import Nat, lt_and_lte
from data.basic.relation_basic import is_symmetric, is_irreflexive
from finite_set import FiniteSet
from data.nat.nat_range_set import range_set, range_set_contains
from graph.simple_graph import SimpleGraph
from graph.simple_graph_degree import neighborhood, neighborhood_contains_eq
from graph.simple_graph_vertex_sets import common_neighborhood, common_neighborhood_contains_eq
from graph.simple_graph_diamond import has_nonadjacent_common_neighbors,
    has_nonadjacent_common_neighbors_intro, is_diamond_free, is_diamond_free_apply
from graph.simple_graph_triangle_free import has_no_common_neighbor,
    has_no_common_neighbor_intro, is_triangle_free, is_triangle_free_intro
from graph.simple_graph_claw import has_independent_neighbor_triple,
    has_independent_neighbor_triple_intro, is_claw_free, is_claw_free_apply

numerals Nat

/// The four vertices `0`, `1`, `2`, `3` on which the diamond and the claw live.
///
/// The vertex type is the naturals and the four-element set is cut out by `range_set`, the same
/// arrangement `src/simple_graph_path.ac` and `src/simple_graph_cycle.ac` use. A dedicated
/// four-element type would work too, but would need its own enumeration lemmas.
let four_vertices: FiniteSet[Nat] = range_set(Nat.4)

/// The four labels are in increasing order.
///
/// Comparisons between numerals are not free, so the ladder is stated once and the distinctness
/// of the labels is read off it.
theorem four_labels_ordered {
    Nat.0 < Nat.1 and Nat.1 < Nat.2 and Nat.2 < Nat.3 and Nat.3 < Nat.4
} by {
    Nat.0.suc = Nat.1
    Nat.0 < Nat.0.suc
    Nat.0 < Nat.1
    Nat.1.suc = Nat.2
    Nat.1 < Nat.1.suc
    Nat.1 < Nat.2
    Nat.2.suc = Nat.3
    Nat.2 < Nat.2.suc
    Nat.2 < Nat.3
    Nat.3.suc = Nat.4
    Nat.3 < Nat.3.suc
    Nat.3 < Nat.4
}

/// Each of the four labels is below the bound, and they are pairwise distinct.
theorem four_labels_lt {
    Nat.0 < Nat.4 and Nat.1 < Nat.4 and Nat.2 < Nat.4 and Nat.3 < Nat.4
} by {
    four_labels_ordered
    (Nat.0 < Nat.1 and Nat.1 < Nat.2 and Nat.2 < Nat.3 and Nat.3 < Nat.4)
    Nat.2 < Nat.3
    Nat.3 < Nat.4
    Nat.3 <= Nat.4
    lt_and_lte(Nat.2, Nat.3, Nat.4)
    Nat.2 < Nat.4
    Nat.1 < Nat.2
    Nat.2 <= Nat.4
    lt_and_lte(Nat.1, Nat.2, Nat.4)
    Nat.1 < Nat.4
    Nat.0 < Nat.1
    Nat.1 <= Nat.4
    lt_and_lte(Nat.0, Nat.1, Nat.4)
    Nat.0 < Nat.4
}

/// The four labels are pairwise distinct.
theorem four_labels_distinct {
    Nat.0 != Nat.1 and Nat.0 != Nat.2 and Nat.0 != Nat.3
        and Nat.1 != Nat.2 and Nat.1 != Nat.3 and Nat.2 != Nat.3
} by {
    four_labels_ordered
    (Nat.0 < Nat.1 and Nat.1 < Nat.2 and Nat.2 < Nat.3 and Nat.3 < Nat.4)
    Nat.0 < Nat.1
    Nat.0 != Nat.1
    Nat.1 < Nat.2
    Nat.1 != Nat.2
    Nat.2 < Nat.3
    Nat.2 != Nat.3
    Nat.1 <= Nat.2
    lt_and_lte(Nat.0, Nat.1, Nat.2)
    Nat.0 < Nat.2
    Nat.0 != Nat.2
    Nat.2 <= Nat.3
    lt_and_lte(Nat.1, Nat.2, Nat.3)
    Nat.1 < Nat.3
    Nat.1 != Nat.3
    lt_and_lte(Nat.0, Nat.1, Nat.3)
    Nat.0 < Nat.3
    Nat.0 != Nat.3
}

/// Each of the four labels is a vertex.
theorem four_vertices_contains(x: Nat) {
    x < Nat.4 implies four_vertices.contains(x)
} by {
    if x < Nat.4 {
        range_set_contains(Nat.4, x)
        range_set(Nat.4).contains(x)
        four_vertices.contains(x)
    }
}

/// Adjacency of the diamond: the complete graph on four vertices with the edge `0`--`3` removed.
define diamond_adj(x: Nat, y: Nat) -> Bool {
    x != y and not (x = Nat.0 and y = Nat.3) and not (x = Nat.3 and y = Nat.0)
}

/// Diamond adjacency is symmetric.
theorem diamond_adj_is_symmetric {
    is_symmetric(diamond_adj)
} by {
    forall(x: Nat, y: Nat) {
        if diamond_adj(x, y) {
            (diamond_adj(x, y) = (x != y and not (x = Nat.0 and y = Nat.3)
                and not (x = Nat.3 and y = Nat.0)))
            x != y
            not (x = Nat.0 and y = Nat.3)
            not (x = Nat.3 and y = Nat.0)
            y != x
            not (y = Nat.3 and x = Nat.0)
            not (y = Nat.0 and x = Nat.3)
            (diamond_adj(y, x) = (y != x and not (y = Nat.0 and x = Nat.3)
                and not (y = Nat.3 and x = Nat.0)))
            diamond_adj(y, x)
        }
        (diamond_adj(x, y) implies diamond_adj(y, x))
    }
}

/// Diamond adjacency has no loops.
theorem diamond_adj_is_irreflexive {
    is_irreflexive(diamond_adj)
} by {
    forall(x: Nat) {
        if diamond_adj(x, x) {
            (diamond_adj(x, x) = (x != x and not (x = Nat.0 and x = Nat.3)
                and not (x = Nat.3 and x = Nat.0)))
            x != x
            false
        }
        not diamond_adj(x, x)
    }
}

/// `diamond_adj` satisfies the simple-graph constraint.
theorem diamond_adj_constraint {
    is_symmetric(diamond_adj) and is_irreflexive(diamond_adj)
} by {
    diamond_adj_is_symmetric
    diamond_adj_is_irreflexive
}

/// The diamond graph exists as a simple graph.
theorem diamond_graph_exists {
    exists(g: SimpleGraph[Nat]) { SimpleGraph.new(diamond_adj) = Option.some(g) }
} by {
    diamond_adj_constraint
}

/// The diamond: four vertices, all adjacent except one pair.
let diamond_graph: SimpleGraph[Nat] satisfy {
    SimpleGraph.new(diamond_adj) = Option.some(diamond_graph)
}

/// Adjacency in the diamond graph.
theorem diamond_graph_adj_iff(x: Nat, y: Nat) {
    diamond_graph.adj(x, y)
        = (x != y and not (x = Nat.0 and y = Nat.3) and not (x = Nat.3 and y = Nat.0))
} by {
    diamond_adj_constraint
    SimpleGraph.new(diamond_adj) = Option.some(diamond_graph)
    diamond_graph.adj = diamond_adj
    diamond_graph.adj(x, y) = diamond_adj(x, y)
    (diamond_adj(x, y) = (x != y and not (x = Nat.0 and y = Nat.3)
        and not (x = Nat.3 and y = Nat.0)))
}

/// The two ends of the missing edge are not adjacent.
theorem diamond_graph_zero_three_not_adj {
    not diamond_graph.adj(Nat.0, Nat.3)
} by {
    diamond_graph_adj_iff(Nat.0, Nat.3)
    (diamond_graph.adj(Nat.0, Nat.3)
        = (Nat.0 != Nat.3 and not (Nat.0 = Nat.0 and Nat.3 = Nat.3)
            and not (Nat.0 = Nat.3 and Nat.3 = Nat.0)))
    (Nat.0 = Nat.0 and Nat.3 = Nat.3)
    not diamond_graph.adj(Nat.0, Nat.3)
}

/// Every other pair of distinct vertices is adjacent.
theorem diamond_graph_adj_of_ne(x: Nat, y: Nat) {
    x != y and not (x = Nat.0 and y = Nat.3) and not (x = Nat.3 and y = Nat.0)
        implies diamond_graph.adj(x, y)
} by {
    if x != y and not (x = Nat.0 and y = Nat.3) and not (x = Nat.3 and y = Nat.0) {
        diamond_graph_adj_iff(x, y)
        (diamond_graph.adj(x, y)
            = (x != y and not (x = Nat.0 and y = Nat.3) and not (x = Nat.3 and y = Nat.0)))
        diamond_graph.adj(x, y)
    }
}

/// The diamond is not diamond free.
///
/// The edge `1`--`2` has the two ends of the missing edge as common neighbours, and those are
/// not adjacent to each other. This is what makes the condition of `src/simple_graph_diamond.ac`
/// non-vacuous: some graph satisfies it and this one does not.
theorem diamond_graph_not_diamond_free {
    not is_diamond_free(diamond_graph, four_vertices)
} by {
    four_labels_lt
    (Nat.0 < Nat.4 and Nat.1 < Nat.4 and Nat.2 < Nat.4 and Nat.3 < Nat.4)
    four_labels_distinct
    (Nat.0 != Nat.1 and Nat.0 != Nat.2 and Nat.0 != Nat.3
        and Nat.1 != Nat.2 and Nat.1 != Nat.3 and Nat.2 != Nat.3)
    four_vertices_contains(Nat.1)
    four_vertices.contains(Nat.1)
    four_vertices_contains(Nat.2)
    four_vertices.contains(Nat.2)
    four_vertices_contains(Nat.0)
    four_vertices.contains(Nat.0)
    four_vertices_contains(Nat.3)
    four_vertices.contains(Nat.3)
    Nat.1 != Nat.2
    diamond_graph_adj_of_ne(Nat.1, Nat.2)
    diamond_graph.adj(Nat.1, Nat.2)
    Nat.1 != Nat.0
    diamond_graph_adj_of_ne(Nat.1, Nat.0)
    diamond_graph.adj(Nat.1, Nat.0)
    Nat.2 != Nat.0
    diamond_graph_adj_of_ne(Nat.2, Nat.0)
    diamond_graph.adj(Nat.2, Nat.0)
    Nat.1 != Nat.3
    diamond_graph_adj_of_ne(Nat.1, Nat.3)
    diamond_graph.adj(Nat.1, Nat.3)
    Nat.2 != Nat.3
    diamond_graph_adj_of_ne(Nat.2, Nat.3)
    diamond_graph.adj(Nat.2, Nat.3)
    common_neighborhood_contains_eq(diamond_graph, four_vertices, Nat.1, Nat.2, Nat.0)
    (common_neighborhood(diamond_graph, four_vertices, Nat.1, Nat.2).contains(Nat.0)
        = (four_vertices.contains(Nat.0) and diamond_graph.adj(Nat.1, Nat.0)
            and diamond_graph.adj(Nat.2, Nat.0)))
    common_neighborhood(diamond_graph, four_vertices, Nat.1, Nat.2).contains(Nat.0)
    common_neighborhood_contains_eq(diamond_graph, four_vertices, Nat.1, Nat.2, Nat.3)
    (common_neighborhood(diamond_graph, four_vertices, Nat.1, Nat.2).contains(Nat.3)
        = (four_vertices.contains(Nat.3) and diamond_graph.adj(Nat.1, Nat.3)
            and diamond_graph.adj(Nat.2, Nat.3)))
    common_neighborhood(diamond_graph, four_vertices, Nat.1, Nat.2).contains(Nat.3)
    Nat.0 != Nat.3
    diamond_graph_zero_three_not_adj
    not diamond_graph.adj(Nat.0, Nat.3)
    has_nonadjacent_common_neighbors_intro(diamond_graph, four_vertices, Nat.1, Nat.2,
        Nat.0, Nat.3)
    has_nonadjacent_common_neighbors(diamond_graph, four_vertices, Nat.1, Nat.2)
    if is_diamond_free(diamond_graph, four_vertices) {
        is_diamond_free_apply(diamond_graph, four_vertices, Nat.1, Nat.2)
        not has_nonadjacent_common_neighbors(diamond_graph, four_vertices, Nat.1, Nat.2)
        false
    }
    not is_diamond_free(diamond_graph, four_vertices)
}

/// Adjacency of the claw: `0` is adjacent to everything else and nothing else is adjacent.
define claw_adj(x: Nat, y: Nat) -> Bool {
    (x = Nat.0 and y != Nat.0) or (y = Nat.0 and x != Nat.0)
}

/// Claw adjacency is symmetric.
theorem claw_adj_is_symmetric {
    is_symmetric(claw_adj)
} by {
    forall(x: Nat, y: Nat) {
        if claw_adj(x, y) {
            (claw_adj(x, y) = ((x = Nat.0 and y != Nat.0) or (y = Nat.0 and x != Nat.0)))
            ((y = Nat.0 and x != Nat.0) or (x = Nat.0 and y != Nat.0))
            (claw_adj(y, x) = ((y = Nat.0 and x != Nat.0) or (x = Nat.0 and y != Nat.0)))
            claw_adj(y, x)
        }
        (claw_adj(x, y) implies claw_adj(y, x))
    }
}

/// Claw adjacency has no loops.
theorem claw_adj_is_irreflexive {
    is_irreflexive(claw_adj)
} by {
    forall(x: Nat) {
        if claw_adj(x, x) {
            (claw_adj(x, x) = ((x = Nat.0 and x != Nat.0) or (x = Nat.0 and x != Nat.0)))
            (x = Nat.0 and x != Nat.0)
            x = Nat.0
            x != Nat.0
            false
        }
        not claw_adj(x, x)
    }
}

/// `claw_adj` satisfies the simple-graph constraint.
theorem claw_adj_constraint {
    is_symmetric(claw_adj) and is_irreflexive(claw_adj)
} by {
    claw_adj_is_symmetric
    claw_adj_is_irreflexive
}

/// The claw graph exists as a simple graph.
theorem claw_graph_exists {
    exists(g: SimpleGraph[Nat]) { SimpleGraph.new(claw_adj) = Option.some(g) }
} by {
    claw_adj_constraint
}

/// The claw: a centre joined to three leaves, with no edges among the leaves.
let claw_graph: SimpleGraph[Nat] satisfy {
    SimpleGraph.new(claw_adj) = Option.some(claw_graph)
}

/// Adjacency in the claw graph.
theorem claw_graph_adj_iff(x: Nat, y: Nat) {
    claw_graph.adj(x, y) = ((x = Nat.0 and y != Nat.0) or (y = Nat.0 and x != Nat.0))
} by {
    claw_adj_constraint
    SimpleGraph.new(claw_adj) = Option.some(claw_graph)
    claw_graph.adj = claw_adj
    claw_graph.adj(x, y) = claw_adj(x, y)
    (claw_adj(x, y) = ((x = Nat.0 and y != Nat.0) or (y = Nat.0 and x != Nat.0)))
}

/// The centre is adjacent to every leaf.
theorem claw_graph_center_adj(y: Nat) {
    y != Nat.0 implies claw_graph.adj(Nat.0, y)
} by {
    if y != Nat.0 {
        claw_graph_adj_iff(Nat.0, y)
        (claw_graph.adj(Nat.0, y)
            = ((Nat.0 = Nat.0 and y != Nat.0) or (y = Nat.0 and Nat.0 != Nat.0)))
        claw_graph.adj(Nat.0, y)
    }
}

/// Two leaves are never adjacent.
theorem claw_graph_leaves_not_adj(x: Nat, y: Nat) {
    x != Nat.0 and y != Nat.0 implies not claw_graph.adj(x, y)
} by {
    if x != Nat.0 and y != Nat.0 {
        claw_graph_adj_iff(x, y)
        (claw_graph.adj(x, y) = ((x = Nat.0 and y != Nat.0) or (y = Nat.0 and x != Nat.0)))
        not (x = Nat.0 and y != Nat.0)
        not (y = Nat.0 and x != Nat.0)
        not claw_graph.adj(x, y)
    }
}

/// The claw is not claw free.
///
/// The centre has the three leaves as neighbours, and no two of them are adjacent. This is the
/// witness that makes the condition of `src/simple_graph_claw.ac` non-vacuous.
theorem claw_graph_not_claw_free {
    not is_claw_free(claw_graph, four_vertices)
} by {
    four_labels_lt
    (Nat.0 < Nat.4 and Nat.1 < Nat.4 and Nat.2 < Nat.4 and Nat.3 < Nat.4)
    four_labels_distinct
    (Nat.0 != Nat.1 and Nat.0 != Nat.2 and Nat.0 != Nat.3
        and Nat.1 != Nat.2 and Nat.1 != Nat.3 and Nat.2 != Nat.3)
    four_vertices_contains(Nat.0)
    four_vertices.contains(Nat.0)
    four_vertices_contains(Nat.1)
    four_vertices.contains(Nat.1)
    four_vertices_contains(Nat.2)
    four_vertices.contains(Nat.2)
    four_vertices_contains(Nat.3)
    four_vertices.contains(Nat.3)
    Nat.1 != Nat.0
    claw_graph_center_adj(Nat.1)
    claw_graph.adj(Nat.0, Nat.1)
    neighborhood_contains_eq(claw_graph, four_vertices, Nat.0, Nat.1)
    (neighborhood(claw_graph, four_vertices, Nat.0).contains(Nat.1)
        = (four_vertices.contains(Nat.1) and claw_graph.adj(Nat.0, Nat.1)))
    neighborhood(claw_graph, four_vertices, Nat.0).contains(Nat.1)
    Nat.2 != Nat.0
    claw_graph_center_adj(Nat.2)
    claw_graph.adj(Nat.0, Nat.2)
    neighborhood_contains_eq(claw_graph, four_vertices, Nat.0, Nat.2)
    (neighborhood(claw_graph, four_vertices, Nat.0).contains(Nat.2)
        = (four_vertices.contains(Nat.2) and claw_graph.adj(Nat.0, Nat.2)))
    neighborhood(claw_graph, four_vertices, Nat.0).contains(Nat.2)
    Nat.3 != Nat.0
    claw_graph_center_adj(Nat.3)
    claw_graph.adj(Nat.0, Nat.3)
    neighborhood_contains_eq(claw_graph, four_vertices, Nat.0, Nat.3)
    (neighborhood(claw_graph, four_vertices, Nat.0).contains(Nat.3)
        = (four_vertices.contains(Nat.3) and claw_graph.adj(Nat.0, Nat.3)))
    neighborhood(claw_graph, four_vertices, Nat.0).contains(Nat.3)
    claw_graph_leaves_not_adj(Nat.1, Nat.2)
    not claw_graph.adj(Nat.1, Nat.2)
    claw_graph_leaves_not_adj(Nat.1, Nat.3)
    not claw_graph.adj(Nat.1, Nat.3)
    claw_graph_leaves_not_adj(Nat.2, Nat.3)
    not claw_graph.adj(Nat.2, Nat.3)
    Nat.1 != Nat.2
    Nat.1 != Nat.3
    Nat.2 != Nat.3
    has_independent_neighbor_triple_intro(claw_graph, four_vertices, Nat.0,
        Nat.1, Nat.2, Nat.3)
    has_independent_neighbor_triple(claw_graph, four_vertices, Nat.0)
    if is_claw_free(claw_graph, four_vertices) {
        is_claw_free_apply(claw_graph, four_vertices, Nat.0)
        not has_independent_neighbor_triple(claw_graph, four_vertices, Nat.0)
        false
    }
    not is_claw_free(claw_graph, four_vertices)
}

/// The claw is triangle free.
///
/// A triangle needs two adjacent leaves, and the only edges are those at the centre. Together
/// with the previous theorem this shows triangle-freeness does not imply claw-freeness, so the
/// two forbidden-subgraph conditions are genuinely independent.
theorem claw_graph_is_triangle_free {
    is_triangle_free(claw_graph, four_vertices)
} by {
    forall(x: Nat, y: Nat) {
        if four_vertices.contains(x) and four_vertices.contains(y)
            and claw_graph.adj(x, y) {
            claw_graph_adj_iff(x, y)
            (claw_graph.adj(x, y)
                = ((x = Nat.0 and y != Nat.0) or (y = Nat.0 and x != Nat.0)))
            forall(z: Nat) {
                if common_neighborhood(claw_graph, four_vertices, x, y).contains(z) {
                    common_neighborhood_contains_eq(claw_graph, four_vertices, x, y, z)
                    (common_neighborhood(claw_graph, four_vertices, x, y).contains(z)
                        = (four_vertices.contains(z) and claw_graph.adj(x, z)
                            and claw_graph.adj(y, z)))
                    claw_graph.adj(x, z)
                    claw_graph.adj(y, z)
                    if x = Nat.0 {
                        y != Nat.0
                        claw_graph_adj_iff(y, z)
                        (claw_graph.adj(y, z)
                            = ((y = Nat.0 and z != Nat.0) or (z = Nat.0 and y != Nat.0)))
                        z = Nat.0
                        claw_graph_adj_iff(Nat.0, Nat.0)
                        (claw_graph.adj(Nat.0, Nat.0)
                            = ((Nat.0 = Nat.0 and Nat.0 != Nat.0)
                                or (Nat.0 = Nat.0 and Nat.0 != Nat.0)))
                        not claw_graph.adj(Nat.0, Nat.0)
                        claw_graph.adj(x, z)
                        false
                    }
                    x != Nat.0
                    y = Nat.0
                    claw_graph_adj_iff(x, z)
                    (claw_graph.adj(x, z)
                        = ((x = Nat.0 and z != Nat.0) or (z = Nat.0 and x != Nat.0)))
                    z = Nat.0
                    claw_graph_adj_iff(Nat.0, Nat.0)
                    (claw_graph.adj(Nat.0, Nat.0)
                        = ((Nat.0 = Nat.0 and Nat.0 != Nat.0)
                            or (Nat.0 = Nat.0 and Nat.0 != Nat.0)))
                    not claw_graph.adj(Nat.0, Nat.0)
                    claw_graph.adj(y, z)
                    false
                }
                not common_neighborhood(claw_graph, four_vertices, x, y).contains(z)
            }
            has_no_common_neighbor_intro(claw_graph, four_vertices, x, y)
            has_no_common_neighbor(claw_graph, four_vertices, x, y)
        }
        (four_vertices.contains(x) and four_vertices.contains(y) and claw_graph.adj(x, y)
            implies has_no_common_neighbor(claw_graph, four_vertices, x, y))
    }
    is_triangle_free_intro(claw_graph, four_vertices)
    is_triangle_free(claw_graph, four_vertices)
}
