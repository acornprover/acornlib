from nat import Nat
from finite_set import FiniteSet, finite_set_subset_contains
from graph.simple_graph import SimpleGraph
from graph.simple_graph_vertex_sets import common_neighborhood, common_neighborhood_contains_eq

numerals Nat

/// True if no vertex of `s` is adjacent to both `x` and `y`.
///
/// Splitting this out of triangle-freeness is what makes the notion usable. Stated
/// directly, triangle-freeness is a three-variable quantifier and neither its
/// introduction nor its restriction to subsets goes through; with the innermost
/// variable named here, both outer statements are two-variable and verify.
define has_no_common_neighbor[V](
    g: SimpleGraph[V], s: FiniteSet[V], x: V, y: V
) -> Bool {
    forall(z: V) {
        not common_neighborhood(g, s, x, y).contains(z)
    }
}

/// No vertex is a common neighbor of a pair without common neighbors.
theorem has_no_common_neighbor_apply[V](
    g: SimpleGraph[V], s: FiniteSet[V], x: V, y: V, z: V
) {
    has_no_common_neighbor(g, s, x, y) implies not common_neighborhood(g, s, x, y).contains(z)
} by {
    if has_no_common_neighbor(g, s, x, y) {
        has_no_common_neighbor(g, s, x, y) = forall(w: V) {
            not common_neighborhood(g, s, x, y).contains(w)
        }
        forall(w: V) {
            not common_neighborhood(g, s, x, y).contains(w)
        }
        not common_neighborhood(g, s, x, y).contains(z)
    }
}

/// A pointwise absence of common neighbors is the absence of a common neighbor.
theorem has_no_common_neighbor_intro[V](
    g: SimpleGraph[V], s: FiniteSet[V], x: V, y: V
) {
    (forall(z: V) { not common_neighborhood(g, s, x, y).contains(z) })
        implies has_no_common_neighbor(g, s, x, y)
} by {
    if forall(z: V) { not common_neighborhood(g, s, x, y).contains(z) } {
        has_no_common_neighbor(g, s, x, y) = forall(w: V) {
            not common_neighborhood(g, s, x, y).contains(w)
        }
        has_no_common_neighbor(g, s, x, y)
    }
}

/// A pair without common neighbors has no vertex adjacent to both.
theorem has_no_common_neighbor_not_adj[V](
    g: SimpleGraph[V], s: FiniteSet[V], x: V, y: V, z: V
) {
    has_no_common_neighbor(g, s, x, y) and s.contains(z) and g.adj(x, z)
        implies not g.adj(y, z)
} by {
    if has_no_common_neighbor(g, s, x, y) and s.contains(z) and g.adj(x, z) {
        if g.adj(y, z) {
            common_neighborhood_contains_eq(g, s, x, y, z)
            common_neighborhood(g, s, x, y).contains(z)
            has_no_common_neighbor_apply(g, s, x, y, z)
            not common_neighborhood(g, s, x, y).contains(z)
            false
        }
        not g.adj(y, z)
    }
}

/// True if no three vertices of `s` are mutually adjacent.
///
/// Phrased as: adjacent vertices have no common neighbor.
define is_triangle_free[V](g: SimpleGraph[V], s: FiniteSet[V]) -> Bool {
    forall(x: V, y: V) {
        s.contains(x) and s.contains(y) and g.adj(x, y)
            implies has_no_common_neighbor(g, s, x, y)
    }
}

/// Adjacent vertices of a triangle-free graph have no common neighbor.
theorem is_triangle_free_apply[V](
    g: SimpleGraph[V], s: FiniteSet[V], x: V, y: V
) {
    is_triangle_free(g, s) and s.contains(x) and s.contains(y) and g.adj(x, y) implies has_no_common_neighbor(g, s, x, y)
} by {
    if is_triangle_free(g, s) and s.contains(x) and s.contains(y) and g.adj(x, y) {
        is_triangle_free(g, s) = forall(u: V, w: V) {
            s.contains(u) and s.contains(w) and g.adj(u, w) implies has_no_common_neighbor(g, s, u, w)
        }
        forall(u: V, w: V) {
            s.contains(u) and s.contains(w) and g.adj(u, w) implies has_no_common_neighbor(g, s, u, w)
        }
        s.contains(x) and s.contains(y) and g.adj(x, y) implies has_no_common_neighbor(g, s, x, y)
        has_no_common_neighbor(g, s, x, y)
    }
}

/// A pointwise no-common-neighbor condition is triangle-freeness.
theorem is_triangle_free_intro[V](g: SimpleGraph[V], s: FiniteSet[V]) {
    (forall(x: V, y: V) {
        s.contains(x) and s.contains(y) and g.adj(x, y) implies has_no_common_neighbor(g, s, x, y)
    }) implies is_triangle_free(g, s)
} by {
    if forall(x: V, y: V) {
        s.contains(x) and s.contains(y) and g.adj(x, y) implies has_no_common_neighbor(g, s, x, y)
    } {
        is_triangle_free(g, s) = forall(u: V, w: V) {
            s.contains(u) and s.contains(w) and g.adj(u, w) implies has_no_common_neighbor(g, s, u, w)
        }
        is_triangle_free(g, s)
    }
}

/// A triangle-free graph has no three mutually adjacent vertices.
///
/// This is the classical reading of the definition, recovered from the
/// common-neighbor form.
theorem is_triangle_free_no_triangle[V](
    g: SimpleGraph[V], s: FiniteSet[V], x: V, y: V, z: V
) {
    is_triangle_free(g, s) and s.contains(x) and s.contains(y) and s.contains(z) and g.adj(x, y) and g.adj(x, z) implies not g.adj(y, z)
} by {
    if is_triangle_free(g, s) and s.contains(x) and s.contains(y) and s.contains(z) and g.adj(x, y) and g.adj(x, z) {
        is_triangle_free_apply(g, s, x, y)
        has_no_common_neighbor(g, s, x, y)
        has_no_common_neighbor_not_adj(g, s, x, y, z)
        not g.adj(y, z)
    }
}
