from nat import Nat
from data.basic.relation_basic import is_symmetric, is_irreflexive
from finite_set import FiniteSet
from graph.simple_graph import SimpleGraph
from graph.simple_graph_vertex_sets import common_neighborhood, common_neighborhood_contains_eq
from graph.simple_graph_triangle_free import has_no_common_neighbor, has_no_common_neighbor_intro,
    is_triangle_free, is_triangle_free_intro

numerals Nat

/// Adjacency on the path: two naturals are adjacent when they are consecutive.
///
/// The vertex type is the naturals rather than a bounded type, and the finite path is cut out
/// by the vertex set that every graph predicate already carries. That keeps the graph itself
/// independent of its length, so one relation serves every `P_n` at once.
define path_adj(x: Nat, y: Nat) -> Bool {
    x.suc = y or y.suc = x
}

/// Consecutiveness is symmetric.
theorem path_adj_is_symmetric {
    is_symmetric(path_adj)
} by {
    forall(x: Nat, y: Nat) {
        if path_adj(x, y) {
            (x.suc = y or y.suc = x)
            (y.suc = x or x.suc = y)
            path_adj(y, x)
        }
        (path_adj(x, y) implies path_adj(y, x))
    }
}

/// No natural is consecutive with itself.
theorem path_adj_is_irreflexive {
    is_irreflexive(path_adj)
} by {
    forall(x: Nat) {
        if path_adj(x, x) {
            (x.suc = x or x.suc = x)
            x.suc = x
            x < x.suc
            x < x
            false
        }
        not path_adj(x, x)
    }
}

/// `path_adj` satisfies the simple-graph constraint.
theorem path_adj_constraint {
    is_symmetric(path_adj) and is_irreflexive(path_adj)
} by {
    path_adj_is_symmetric
    path_adj_is_irreflexive
}

/// The path graph on the naturals.
///
/// A finite path `P_n` is this graph restricted to the vertex set `{0, ..., n - 1}`.
/// The path graph exists as a simple graph.
theorem path_graph_exists {
    exists(g: SimpleGraph[Nat]) { SimpleGraph.new(path_adj) = Option.some(g) }
} by {
    path_adj_constraint
}

/// The infinite path graph on the naturals.
let path_graph: SimpleGraph[Nat] satisfy {
    SimpleGraph.new(path_adj) = Option.some(path_graph)
}

/// Adjacency in the path graph is consecutiveness.
theorem path_graph_adj_iff(x: Nat, y: Nat) {
    path_graph.adj(x, y) = (x.suc = y or y.suc = x)
} by {
    path_adj_constraint
    SimpleGraph.new(path_adj) = Option.some(path_graph)
    path_graph.adj = path_adj
    path_graph.adj(x, y) = path_adj(x, y)
    path_adj(x, y) = (x.suc = y or y.suc = x)
}

/// A vertex is adjacent to its successor.
theorem path_graph_adj_suc(x: Nat) {
    path_graph.adj(x, x.suc)
} by {
    path_graph_adj_iff(x, x.suc)
    path_graph.adj(x, x.suc) = (x.suc = x.suc or x.suc.suc = x)
}

/// Adjacent vertices in the path graph are distinct.
theorem path_graph_adj_ne(x: Nat, y: Nat) {
    path_graph.adj(x, y) implies x != y
} by {
    if path_graph.adj(x, y) {
        if x = y {
            path_graph.adj(x, x)
            path_adj_is_irreflexive
            not path_adj(x, x)
            path_graph_adj_iff(x, x)
            path_graph.adj(x, x) = (x.suc = x or x.suc = x)
            x.suc = x
            x < x.suc
            x < x
            false
        }
        x != y
    }
}

/// The two neighbors of a vertex are its predecessor and its successor.
///
/// Stated as: anything adjacent to `x` is either `x + 1`, or is a vertex whose successor is
/// `x`. The second case is not written as `x - 1` because truncated subtraction would make the
/// statement false at `x = 0`, where the vertex has no predecessor.
theorem path_graph_neighbor_cases(x: Nat, y: Nat) {
    path_graph.adj(x, y) implies (y = x.suc or y.suc = x)
} by {
    if path_graph.adj(x, y) {
        path_graph_adj_iff(x, y)
        (x.suc = y or y.suc = x)
        (y = x.suc or y.suc = x)
    }
}

/// Two vertices adjacent to a common vertex cannot themselves be adjacent.
///
/// The two neighbors of `x` are `x - 1` and `x + 1`, which differ by two, so they are not
/// consecutive. This is the whole content of the path being triangle free.
theorem path_graph_neighbors_not_adj(x: Nat, y: Nat, z: Nat) {
    path_graph.adj(x, y) and path_graph.adj(x, z) and y != z
        implies not path_graph.adj(y, z)
} by {
    if path_graph.adj(x, y) and path_graph.adj(x, z) and y != z {
        path_graph_neighbor_cases(x, y)
        (y = x.suc or y.suc = x)
        path_graph_neighbor_cases(x, z)
        (z = x.suc or z.suc = x)
        if y = x.suc {
            z != x.suc
            z.suc = x
            if path_graph.adj(y, z) {
                path_graph_adj_iff(y, z)
                (y.suc = z or z.suc = y)
                if y.suc = z {
                    x.suc.suc = z
                    z.suc = x
                    x.suc.suc.suc = x
                    x < x.suc
                    x.suc < x.suc.suc
                    x.suc.suc < x.suc.suc.suc
                    x < x.suc.suc.suc
                    x < x
                    false
                }
                z.suc = y
                x = y
                x = x.suc
                x < x.suc
                x < x
                false
            }
            not path_graph.adj(y, z)
        }
        if y != x.suc {
            y.suc = x
            if z = x.suc {
                if path_graph.adj(y, z) {
                    path_graph_adj_iff(y, z)
                    (y.suc = z or z.suc = y)
                    if y.suc = z {
                        x = z
                        x = x.suc
                        x < x.suc
                        x < x
                        false
                    }
                    z.suc = y
                    x.suc.suc = y
                    y.suc = x
                    x.suc.suc.suc = x
                    x < x.suc
                    x.suc < x.suc.suc
                    x.suc.suc < x.suc.suc.suc
                    x < x.suc.suc.suc
                    x < x
                    false
                }
                not path_graph.adj(y, z)
            }
            if z != x.suc {
                z.suc = x
                y.suc = x
                y = z
                false
            }
            not path_graph.adj(y, z)
        }
        not path_graph.adj(y, z)
    }
}

/// Adjacent vertices of the path have no common neighbor.
///
/// A common neighbor of `x` and `y` would make `x` and `y` two distinct neighbors of a single
/// vertex, and those differ by two rather than being adjacent.
theorem path_graph_has_no_common_neighbor(s: FiniteSet[Nat], x: Nat, y: Nat) {
    path_graph.adj(x, y) implies has_no_common_neighbor(path_graph, s, x, y)
} by {
    if path_graph.adj(x, y) {
        path_graph_adj_ne(x, y)
        x != y
        forall(z: Nat) {
            if common_neighborhood(path_graph, s, x, y).contains(z) {
                common_neighborhood_contains_eq(path_graph, s, x, y, z)
                (s.contains(z) and path_graph.adj(x, z) and path_graph.adj(y, z))
                path_graph.adj(z, x)
                path_graph.adj(z, y)
                path_graph_neighbors_not_adj(z, x, y)
                not path_graph.adj(x, y)
                false
            }
            not common_neighborhood(path_graph, s, x, y).contains(z)
        }
        has_no_common_neighbor_intro(path_graph, s, x, y)
        has_no_common_neighbor(path_graph, s, x, y)
    }
}

/// Every finite path is triangle free.
///
/// A triangle would need two adjacent vertices with a common neighbor, and the two neighbors
/// of any vertex differ by two.
theorem path_graph_is_triangle_free(s: FiniteSet[Nat]) {
    is_triangle_free(path_graph, s)
} by {
    forall(x: Nat, y: Nat) {
        if s.contains(x) and s.contains(y) and path_graph.adj(x, y) {
            path_graph_has_no_common_neighbor(s, x, y)
            has_no_common_neighbor(path_graph, s, x, y)
        }
        (s.contains(x) and s.contains(y) and path_graph.adj(x, y)
            implies has_no_common_neighbor(path_graph, s, x, y))
    }
    is_triangle_free_intro(path_graph, s)
}
