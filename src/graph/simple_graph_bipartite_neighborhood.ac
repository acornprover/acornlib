from nat import Nat
from finite_set import FiniteSet
from graph.simple_graph import SimpleGraph
from graph.simple_graph_degree import neighborhood, neighborhood_contains_eq
from graph.simple_graph_bipartite import is_bipartition, is_bipartition_apply
from graph.simple_graph_bipartite_sides import bipartite_side, bipartite_side_contains_eq,
    opposite_part, opposite_part_eq

numerals Nat

/// Every neighbor of a vertex on one side lies on the other side.
///
/// This is the bipartition condition read through the neighborhood: adjacency
/// forces opposite colours, so a neighbor of a coloured vertex has the other colour.
theorem bipartite_neighborhood_opposite_side[V](
    g: SimpleGraph[V], s: FiniteSet[V], part: V -> Bool, v: V, w: V
) {
    is_bipartition(g, s, part) and s.contains(v) and part(v) and neighborhood(g, s, v).contains(w) implies bipartite_side(s, opposite_part(part)).contains(w)
} by {
    if is_bipartition(g, s, part) and s.contains(v) and part(v) and neighborhood(g, s, v).contains(w) {
        neighborhood_contains_eq(g, s, v, w)
        s.contains(w) and g.adj(v, w)
        is_bipartition_apply(g, s, part, v, w)
        part(v) != part(w)
        not part(w)
        opposite_part_eq(part, w)
        opposite_part(part)(w) = not part(w)
        opposite_part(part)(w)
        bipartite_side_contains_eq(s, opposite_part(part), w)
        bipartite_side(s, opposite_part(part)).contains(w)
    }
}

/// Every neighbor of a vertex on the second side lies on the first.
theorem bipartite_neighborhood_first_side[V](
    g: SimpleGraph[V], s: FiniteSet[V], part: V -> Bool, v: V, w: V
) {
    is_bipartition(g, s, part) and s.contains(v) and not part(v) and neighborhood(g, s, v).contains(w) implies bipartite_side(s, part).contains(w)
} by {
    if is_bipartition(g, s, part) and s.contains(v) and not part(v) and neighborhood(g, s, v).contains(w) {
        neighborhood_contains_eq(g, s, v, w)
        s.contains(w) and g.adj(v, w)
        is_bipartition_apply(g, s, part, v, w)
        part(v) != part(w)
        part(w)
        bipartite_side_contains_eq(s, part, w)
        bipartite_side(s, part).contains(w)
    }
}

/// A vertex has no neighbor on its own side.
theorem bipartite_no_neighbor_same_side[V](
    g: SimpleGraph[V], s: FiniteSet[V], part: V -> Bool, v: V, w: V
) {
    is_bipartition(g, s, part) and s.contains(v) and part(v) and bipartite_side(s, part).contains(w) implies not neighborhood(g, s, v).contains(w)
} by {
    if is_bipartition(g, s, part) and s.contains(v) and part(v) and bipartite_side(s, part).contains(w) {
        bipartite_side_contains_eq(s, part, w)
        s.contains(w) and part(w)
        if neighborhood(g, s, v).contains(w) {
            neighborhood_contains_eq(g, s, v, w)
            g.adj(v, w)
            is_bipartition_apply(g, s, part, v, w)
            part(v) != part(w)
            false
        }
        not neighborhood(g, s, v).contains(w)
    }
}

/// The neighborhood of a vertex avoids that vertex's own side entirely.
theorem bipartite_neighborhood_disjoint_from_side[V](
    g: SimpleGraph[V], s: FiniteSet[V], part: V -> Bool, v: V, w: V
) {
    is_bipartition(g, s, part) and s.contains(v) and part(v) implies not (neighborhood(g, s, v).contains(w) and bipartite_side(s, part).contains(w))
} by {
    if is_bipartition(g, s, part) and s.contains(v) and part(v) {
        if neighborhood(g, s, v).contains(w) and bipartite_side(s, part).contains(w) {
            bipartite_no_neighbor_same_side(g, s, part, v, w)
            not neighborhood(g, s, v).contains(w)
            false
        }
        not (neighborhood(g, s, v).contains(w) and bipartite_side(s, part).contains(w))
    }
}
