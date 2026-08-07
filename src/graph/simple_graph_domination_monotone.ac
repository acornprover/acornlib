from nat import Nat, lte_trans
from finite_set import FiniteSet, finite_set_subset_contains
from data.finite.finite_set_card import fs_card
from graph.simple_graph import SimpleGraph, complete_graph, complete_graph_adj_iff_ne,
    simple_graph_adj_ne
from graph.simple_graph_domination import is_dominating_set, is_dominating_set_apply,
    is_dominating_set_intro, is_dominated, is_dominated_of_contains, is_dominated_of_adj,
    has_neighbor_in, has_neighbor_in_witness
from graph.simple_graph_domination_number import domination_number, domination_number_is_least
from graph.simple_graph_minimum_dominating import is_minimum_dominating_set,
    is_minimum_dominating_set_apply, is_dominating_subset, is_dominating_subset_apply,
    minimum_dominating_set_exists

numerals Nat

/// True if every edge of `g` is also an edge of `h`.
///
/// Both graphs are on the same vertex type, so this is the relation of adding edges without
/// touching the vertices. Its point is that domination can only get easier.
define has_all_edges_of[V](h: SimpleGraph[V], g: SimpleGraph[V]) -> Bool {
    forall(x: V, y: V) {
        g.adj(x, y) implies h.adj(x, y)
    }
}

/// An edge of the smaller graph is an edge of the larger.
theorem has_all_edges_of_apply[V](
    h: SimpleGraph[V], g: SimpleGraph[V], x: V, y: V
) {
    has_all_edges_of(h, g) and g.adj(x, y) implies h.adj(x, y)
} by {
    if has_all_edges_of(h, g) and g.adj(x, y) {
        has_all_edges_of(h, g) = forall(a: V, b: V) {
            g.adj(a, b) implies h.adj(a, b)
        }
        forall(a: V, b: V) {
            g.adj(a, b) implies h.adj(a, b)
        }
        (g.adj(x, y) implies h.adj(x, y))
        h.adj(x, y)
    }
}

/// A pointwise edge containment gives the relation.
theorem has_all_edges_of_intro[V](h: SimpleGraph[V], g: SimpleGraph[V]) {
    (forall(x: V, y: V) { g.adj(x, y) implies h.adj(x, y) }) implies has_all_edges_of(h, g)
} by {
    if forall(x: V, y: V) { g.adj(x, y) implies h.adj(x, y) } {
        has_all_edges_of(h, g) = forall(a: V, b: V) {
            g.adj(a, b) implies h.adj(a, b)
        }
        has_all_edges_of(h, g)
    }
}

/// The complete graph has all the edges of every graph.
///
/// Adjacency in a simple graph forces distinct endpoints, which is exactly adjacency in the
/// complete graph.
theorem complete_graph_has_all_edges[V](g: SimpleGraph[V]) {
    has_all_edges_of(complete_graph[V], g)
} by {
    forall(x: V, y: V) {
        if g.adj(x, y) {
            simple_graph_adj_ne(g, x, y)
            x != y
            complete_graph_adj_iff_ne(x, y)
            complete_graph[V].adj(x, y) = (x != y)
            complete_graph[V].adj(x, y)
        }
        (g.adj(x, y) implies complete_graph[V].adj(x, y))
    }
    has_all_edges_of_intro(complete_graph[V], g)
    has_all_edges_of(complete_graph[V], g)
}

/// Adding edges preserves domination of a single vertex.
theorem is_dominated_of_has_all_edges[V](
    h: SimpleGraph[V], g: SimpleGraph[V], d: FiniteSet[V], v: V
) {
    has_all_edges_of(h, g) and is_dominated(g, d, v) implies is_dominated(h, d, v)
} by {
    if has_all_edges_of(h, g) and is_dominated(g, d, v) {
        if d.contains(v) {
            is_dominated_of_contains(h, d, v)
            is_dominated(h, d, v)
        }
        if not d.contains(v) {
            has_neighbor_in(g, d, v)
            has_neighbor_in_witness(g, d, v)
            let (w: V) satisfy {
                d.contains(w) and g.adj(w, v)
            }
            has_all_edges_of_apply(h, g, w, v)
            h.adj(w, v)
            is_dominated_of_adj(h, d, v, w)
            is_dominated(h, d, v)
        }
        is_dominated(h, d, v)
    }
}

/// A dominating set stays dominating when edges are added.
theorem is_dominating_set_of_has_all_edges[V](
    h: SimpleGraph[V], g: SimpleGraph[V], s: FiniteSet[V], d: FiniteSet[V]
) {
    has_all_edges_of(h, g) and is_dominating_set(g, s, d) implies is_dominating_set(h, s, d)
} by {
    if has_all_edges_of(h, g) and is_dominating_set(g, s, d) {
        forall(v: V) {
            if s.contains(v) {
                is_dominating_set_apply(g, s, d, v)
                is_dominated(g, d, v)
                is_dominated_of_has_all_edges(h, g, d, v)
                is_dominated(h, d, v)
            }
            s.contains(v) implies is_dominated(h, d, v)
        }
        is_dominating_set_intro(h, s, d)
        is_dominating_set(h, s, d)
    }
}

/// Adding edges cannot raise the domination number.
///
/// A minimum dominating set of the sparser graph still dominates the denser one, so it
/// bounds the denser graph's domination number.
theorem domination_number_antitone_in_edges[V](
    h: SimpleGraph[V], g: SimpleGraph[V], s: FiniteSet[V]
) {
    has_all_edges_of(h, g) implies domination_number(h, s) <= domination_number(g, s)
} by {
    if has_all_edges_of(h, g) {
        minimum_dominating_set_exists(g, s)
        let (d: FiniteSet[V]) satisfy {
            is_minimum_dominating_set(g, s, d)
        }
        is_minimum_dominating_set_apply(g, s, d)
        is_dominating_subset(g, s, d)
        fs_card(d) = domination_number(g, s)
        is_dominating_subset_apply(g, s, d)
        d.subset_eq(s)
        is_dominating_set(g, s, d)
        is_dominating_set_of_has_all_edges(h, g, s, d)
        is_dominating_set(h, s, d)
        domination_number_is_least(h, s, d)
        domination_number(h, s) <= fs_card(d)
        domination_number(h, s) <= domination_number(g, s)
    }
}

/// No graph has a smaller domination number than the complete graph.
theorem complete_graph_domination_number_least[V](g: SimpleGraph[V], s: FiniteSet[V]) {
    domination_number(complete_graph[V], s) <= domination_number(g, s)
} by {
    complete_graph_has_all_edges(g)
    has_all_edges_of(complete_graph[V], g)
    domination_number_antitone_in_edges(complete_graph[V], g, s)
    domination_number(complete_graph[V], s) <= domination_number(g, s)
}

/// The relation is transitive, so the bound composes across a chain of graphs.
theorem has_all_edges_of_trans[V](
    k: SimpleGraph[V], h: SimpleGraph[V], g: SimpleGraph[V]
) {
    has_all_edges_of(k, h) and has_all_edges_of(h, g) implies has_all_edges_of(k, g)
} by {
    if has_all_edges_of(k, h) and has_all_edges_of(h, g) {
        forall(x: V, y: V) {
            if g.adj(x, y) {
                has_all_edges_of_apply(h, g, x, y)
                h.adj(x, y)
                has_all_edges_of_apply(k, h, x, y)
                k.adj(x, y)
            }
            (g.adj(x, y) implies k.adj(x, y))
        }
        has_all_edges_of_intro(k, g)
        has_all_edges_of(k, g)
    }
}
