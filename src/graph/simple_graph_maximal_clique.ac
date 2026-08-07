from finite_set import FiniteSet, fs_union, finite_set_union_contains_eq,
    finite_set_subset_contains, finite_set_subset_union_left, finite_set_subset_union_right
from data.finite.finite_set_subset_intro import fs_subset_eq_intro
from graph.simple_graph import SimpleGraph, simple_graph_adj_comm
from graph.simple_graph_vertex_sets import common_neighborhood, common_neighborhood_contains_eq
from graph.simple_graph_independent import is_clique_in, is_clique_in_apply, is_clique_in_intro
from graph.simple_graph_diamond import has_nonadjacent_common_neighbors,
    has_nonadjacent_common_neighbors_intro, is_diamond_free, is_diamond_free_apply

/// True if `c` is a clique of `s` that no larger clique of `s` contains.
///
/// Maximality under inclusion rather than by size: a maximal clique need not be a largest one.
/// This is the notion the diamond-free theorem below is about.
define is_maximal_clique_in[V](
    g: SimpleGraph[V], s: FiniteSet[V], c: FiniteSet[V]
) -> Bool {
    c.subset_eq(s) and is_clique_in(g, c) and forall(d: FiniteSet[V]) {
        c.subset_eq(d) and d.subset_eq(s) and is_clique_in(g, d) implies d = c
    }
}

/// A maximal clique is a clique of the ambient set.
theorem is_maximal_clique_in_clique[V](
    g: SimpleGraph[V], s: FiniteSet[V], c: FiniteSet[V]
) {
    is_maximal_clique_in(g, s, c) implies c.subset_eq(s) and is_clique_in(g, c)
} by {
    if is_maximal_clique_in(g, s, c) {
        (is_maximal_clique_in(g, s, c)
            = (c.subset_eq(s) and is_clique_in(g, c) and forall(d: FiniteSet[V]) {
                c.subset_eq(d) and d.subset_eq(s) and is_clique_in(g, d) implies d = c
            }))
        c.subset_eq(s)
        is_clique_in(g, c)
        (c.subset_eq(s) and is_clique_in(g, c))
    }
}

/// Nothing larger than a maximal clique is a clique.
theorem is_maximal_clique_in_apply[V](
    g: SimpleGraph[V], s: FiniteSet[V], c: FiniteSet[V], d: FiniteSet[V]
) {
    is_maximal_clique_in(g, s, c) and c.subset_eq(d) and d.subset_eq(s)
        and is_clique_in(g, d)
        implies d = c
} by {
    if is_maximal_clique_in(g, s, c) and c.subset_eq(d) and d.subset_eq(s)
        and is_clique_in(g, d) {
        (is_maximal_clique_in(g, s, c)
            = (c.subset_eq(s) and is_clique_in(g, c) and forall(e: FiniteSet[V]) {
                c.subset_eq(e) and e.subset_eq(s) and is_clique_in(g, e) implies e = c
            }))
        forall(e: FiniteSet[V]) {
            c.subset_eq(e) and e.subset_eq(s) and is_clique_in(g, e) implies e = c
        }
        (c.subset_eq(d) and d.subset_eq(s) and is_clique_in(g, d) implies d = c)
        d = c
    }
}

/// A vertex of one clique and a vertex of another are adjacent, given a shared edge.
///
/// Both are common neighbours of the shared edge, since neither is one of its ends, and
/// diamond-freeness forbids two common neighbours of an edge from being non-adjacent.
theorem diamond_free_cross_adj[V](
    g: SimpleGraph[V], s: FiniteSet[V], c: FiniteSet[V], e: FiniteSet[V],
    x: V, y: V, u: V, v: V
) {
    is_diamond_free(g, s) and g.adj(x, y)
        and c.subset_eq(s) and is_clique_in(g, c) and c.contains(x) and c.contains(y)
        and e.subset_eq(s) and is_clique_in(g, e) and e.contains(x) and e.contains(y)
        and c.contains(u) and e.contains(v)
        and u != x and u != y and v != x and v != y and u != v
        implies g.adj(u, v)
} by {
    if is_diamond_free(g, s) and g.adj(x, y)
        and c.subset_eq(s) and is_clique_in(g, c) and c.contains(x) and c.contains(y)
        and e.subset_eq(s) and is_clique_in(g, e) and e.contains(x) and e.contains(y)
        and c.contains(u) and e.contains(v)
        and u != x and u != y and v != x and v != y and u != v {
        finite_set_subset_contains(c, s, x)
        s.contains(x)
        finite_set_subset_contains(c, s, y)
        s.contains(y)
        is_clique_in_apply(g, c, u, x)
        g.adj(u, x)
        simple_graph_adj_comm(g, u, x)
        (g.adj(u, x) = g.adj(x, u))
        g.adj(x, u)
        is_clique_in_apply(g, c, u, y)
        g.adj(u, y)
        simple_graph_adj_comm(g, u, y)
        (g.adj(u, y) = g.adj(y, u))
        g.adj(y, u)
        is_clique_in_apply(g, e, v, x)
        g.adj(v, x)
        simple_graph_adj_comm(g, v, x)
        (g.adj(v, x) = g.adj(x, v))
        g.adj(x, v)
        is_clique_in_apply(g, e, v, y)
        g.adj(v, y)
        simple_graph_adj_comm(g, v, y)
        (g.adj(v, y) = g.adj(y, v))
        g.adj(y, v)
        finite_set_subset_contains(c, s, u)
        s.contains(u)
        finite_set_subset_contains(e, s, v)
        s.contains(v)
        common_neighborhood_contains_eq(g, s, x, y, u)
        (common_neighborhood(g, s, x, y).contains(u)
            = (s.contains(u) and g.adj(x, u) and g.adj(y, u)))
        common_neighborhood(g, s, x, y).contains(u)
        common_neighborhood_contains_eq(g, s, x, y, v)
        (common_neighborhood(g, s, x, y).contains(v)
            = (s.contains(v) and g.adj(x, v) and g.adj(y, v)))
        common_neighborhood(g, s, x, y).contains(v)
        if not g.adj(u, v) {
            has_nonadjacent_common_neighbors_intro(g, s, x, y, u, v)
            has_nonadjacent_common_neighbors(g, s, x, y)
            is_diamond_free_apply(g, s, x, y)
            not has_nonadjacent_common_neighbors(g, s, x, y)
            false
        }
        g.adj(u, v)
    }
}

/// Two cliques sharing an edge have a clique union, in a diamond-free graph.
///
/// Vertices in the same clique are adjacent outright. A vertex of one clique and a vertex of
/// the other are handled by the cross lemma, in whichever of the two orientations applies.
theorem diamond_free_clique_union[V](
    g: SimpleGraph[V], s: FiniteSet[V], c: FiniteSet[V], e: FiniteSet[V], x: V, y: V
) {
    is_diamond_free(g, s) and g.adj(x, y)
        and c.subset_eq(s) and is_clique_in(g, c) and c.contains(x) and c.contains(y)
        and e.subset_eq(s) and is_clique_in(g, e) and e.contains(x) and e.contains(y)
        implies is_clique_in(g, fs_union(c, e))
} by {
    if is_diamond_free(g, s) and g.adj(x, y)
        and c.subset_eq(s) and is_clique_in(g, c) and c.contains(x) and c.contains(y)
        and e.subset_eq(s) and is_clique_in(g, e) and e.contains(x) and e.contains(y) {
        forall(u: V, v: V) {
            if fs_union(c, e).contains(u) and fs_union(c, e).contains(v) and u != v {
                finite_set_union_contains_eq(c, e, u)
                (fs_union(c, e).contains(u) = (c.contains(u) or e.contains(u)))
                finite_set_union_contains_eq(c, e, v)
                (fs_union(c, e).contains(v) = (c.contains(v) or e.contains(v)))
                if c.contains(u) and c.contains(v) {
                    is_clique_in_apply(g, c, u, v)
                    g.adj(u, v)
                }
                if e.contains(u) and e.contains(v) {
                    is_clique_in_apply(g, e, u, v)
                    g.adj(u, v)
                }
                if not (c.contains(u) and c.contains(v)) {
                    if not (e.contains(u) and e.contains(v)) {
                        if u = x {
                            e.contains(u)
                            e.contains(v)
                            false
                        }
                        u != x
                        if u = y {
                            e.contains(u)
                            e.contains(v)
                            false
                        }
                        u != y
                        if v = x {
                            c.contains(v)
                            c.contains(u)
                            false
                        }
                        v != x
                        if v = y {
                            c.contains(v)
                            c.contains(u)
                            false
                        }
                        v != y
                        if c.contains(u) {
                            not c.contains(v)
                            e.contains(v)
                            diamond_free_cross_adj(g, s, c, e, x, y, u, v)
                            g.adj(u, v)
                        }
                        if not c.contains(u) {
                            e.contains(u)
                            not e.contains(v)
                            c.contains(v)
                            diamond_free_cross_adj(g, s, c, e, x, y, v, u)
                            g.adj(v, u)
                            simple_graph_adj_comm(g, v, u)
                            (g.adj(v, u) = g.adj(u, v))
                            g.adj(u, v)
                        }
                        g.adj(u, v)
                    }
                    g.adj(u, v)
                }
                g.adj(u, v)
            }
            (fs_union(c, e).contains(u) and fs_union(c, e).contains(v) and u != v
                implies g.adj(u, v))
        }
        is_clique_in_intro(g, fs_union(c, e))
        is_clique_in(g, fs_union(c, e))
    }
}

/// In a diamond-free graph an edge lies in at most one maximal clique.
///
/// The union of two maximal cliques through the same edge is again a clique, so maximality
/// forces each of them to be that union. This is the structural characterisation of
/// diamond-freeness recorded in `src/simple_graph_diamond.ac`.
theorem diamond_free_maximal_clique_unique[V](
    g: SimpleGraph[V], s: FiniteSet[V], c: FiniteSet[V], e: FiniteSet[V], x: V, y: V
) {
    is_diamond_free(g, s) and g.adj(x, y)
        and is_maximal_clique_in(g, s, c) and c.contains(x) and c.contains(y)
        and is_maximal_clique_in(g, s, e) and e.contains(x) and e.contains(y)
        implies c = e
} by {
    if is_diamond_free(g, s) and g.adj(x, y)
        and is_maximal_clique_in(g, s, c) and c.contains(x) and c.contains(y)
        and is_maximal_clique_in(g, s, e) and e.contains(x) and e.contains(y) {
        is_maximal_clique_in_clique(g, s, c)
        (c.subset_eq(s) and is_clique_in(g, c))
        is_maximal_clique_in_clique(g, s, e)
        (e.subset_eq(s) and is_clique_in(g, e))
        diamond_free_clique_union(g, s, c, e, x, y)
        is_clique_in(g, fs_union(c, e))
        forall(z: V) {
            if fs_union(c, e).contains(z) {
                finite_set_union_contains_eq(c, e, z)
                (fs_union(c, e).contains(z) = (c.contains(z) or e.contains(z)))
                if c.contains(z) {
                    finite_set_subset_contains(c, s, z)
                    s.contains(z)
                }
                if not c.contains(z) {
                    e.contains(z)
                    finite_set_subset_contains(e, s, z)
                    s.contains(z)
                }
                s.contains(z)
            }
            (fs_union(c, e).contains(z) implies s.contains(z))
        }
        fs_subset_eq_intro(fs_union(c, e), s)
        fs_union(c, e).subset_eq(s)
        finite_set_subset_union_left(c, e)
        c.subset_eq(fs_union(c, e))
        is_maximal_clique_in_apply(g, s, c, fs_union(c, e))
        fs_union(c, e) = c
        finite_set_subset_union_right(c, e)
        e.subset_eq(fs_union(c, e))
        is_maximal_clique_in_apply(g, s, e, fs_union(c, e))
        fs_union(c, e) = e
        c = e
    }
}
