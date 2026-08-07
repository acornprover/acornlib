from nat import Nat
from finite_set import FiniteSet, finite_set_subset_contains
from data.basic.set import Set
from graph.simple_graph import SimpleGraph, induced_subgraph, induced_subgraph_adj_eq
from graph.simple_graph_independent import is_independent_in, is_independent_in_apply,
    is_independent_in_intro, is_clique_in, is_clique_in_apply, is_clique_in_intro
from graph.simple_graph_bipartite import is_bipartition, is_bipartition_apply, is_bipartition_intro
from graph.simple_graph_vertex_sets import is_vertex_cover, is_vertex_cover_apply,
    is_vertex_cover_intro

numerals Nat

/// Adjacency in the subgraph induced on a finite vertex set.
theorem finite_induced_adj_eq[V](g: SimpleGraph[V], s: FiniteSet[V], x: V, y: V) {
    induced_subgraph(g, s.underlying_set).adj(x, y) = (g.adj(x, y) and s.contains(x) and s.contains(y))
} by {
    induced_subgraph_adj_eq(g, s.underlying_set, x, y)
    induced_subgraph(g, s.underlying_set).adj(x, y) = (g.adj(x, y) and s.underlying_set.contains(x) and s.underlying_set.contains(y))
    s.contains(x) = s.underlying_set.contains(x)
    s.contains(y) = s.underlying_set.contains(y)
}

/// Adjacency in an induced subgraph implies adjacency in the whole graph.
theorem finite_induced_adj_imp[V](g: SimpleGraph[V], s: FiniteSet[V], x: V, y: V) {
    induced_subgraph(g, s.underlying_set).adj(x, y) implies g.adj(x, y)
} by {
    finite_induced_adj_eq(g, s, x, y)
}

/// Adjacent vertices of an induced subgraph lie in the inducing set.
theorem finite_induced_adj_contains[V](g: SimpleGraph[V], s: FiniteSet[V], x: V, y: V) {
    induced_subgraph(g, s.underlying_set).adj(x, y) implies s.contains(x) and s.contains(y)
} by {
    finite_induced_adj_eq(g, s, x, y)
}

/// Independence is inherited by every induced subgraph.
theorem is_independent_in_induced[V](g: SimpleGraph[V], s: FiniteSet[V], t: FiniteSet[V]) {
    is_independent_in(g, t) implies is_independent_in(induced_subgraph(g, s.underlying_set), t)
} by {
    if is_independent_in(g, t) {
        forall(x: V, y: V) {
            if t.contains(x) and t.contains(y) {
                is_independent_in_apply(g, t, x, y)
                not g.adj(x, y)
                finite_induced_adj_eq(g, s, x, y)
                not induced_subgraph(g, s.underlying_set).adj(x, y)
            }
            t.contains(x) and t.contains(y) implies not induced_subgraph(g, s.underlying_set).adj(x, y)
        }
        is_independent_in_intro(induced_subgraph(g, s.underlying_set), t)
        is_independent_in(induced_subgraph(g, s.underlying_set), t)
    }
}

/// A clique of an induced subgraph is a clique of the whole graph.
theorem is_clique_in_of_induced[V](g: SimpleGraph[V], s: FiniteSet[V], t: FiniteSet[V]) {
    is_clique_in(induced_subgraph(g, s.underlying_set), t) implies is_clique_in(g, t)
} by {
    if is_clique_in(induced_subgraph(g, s.underlying_set), t) {
        forall(x: V, y: V) {
            if t.contains(x) and t.contains(y) and x != y {
                is_clique_in_apply(induced_subgraph(g, s.underlying_set), t, x, y)
                induced_subgraph(g, s.underlying_set).adj(x, y)
                finite_induced_adj_imp(g, s, x, y)
                g.adj(x, y)
            }
            t.contains(x) and t.contains(y) and x != y implies g.adj(x, y)
        }
        is_clique_in_intro(g, t)
        is_clique_in(g, t)
    }
}

/// A clique contained in the inducing set stays a clique in the induced subgraph.
theorem is_clique_in_induced_of_subset[V](
    g: SimpleGraph[V], s: FiniteSet[V], t: FiniteSet[V]
) {
    t.subset_eq(s) and is_clique_in(g, t) implies is_clique_in(induced_subgraph(g, s.underlying_set), t)
} by {
    if t.subset_eq(s) and is_clique_in(g, t) {
        forall(x: V, y: V) {
            if t.contains(x) and t.contains(y) and x != y {
                is_clique_in_apply(g, t, x, y)
                g.adj(x, y)
                finite_set_subset_contains(t, s, x)
                finite_set_subset_contains(t, s, y)
                s.contains(x)
                s.contains(y)
                finite_induced_adj_eq(g, s, x, y)
                induced_subgraph(g, s.underlying_set).adj(x, y)
            }
            t.contains(x) and t.contains(y) and x != y implies induced_subgraph(g, s.underlying_set).adj(x, y)
        }
        is_clique_in_intro(induced_subgraph(g, s.underlying_set), t)
        is_clique_in(induced_subgraph(g, s.underlying_set), t)
    }
}

/// A bipartition of a graph is a bipartition of every induced subgraph.
theorem is_bipartition_induced[V](
    g: SimpleGraph[V], s: FiniteSet[V], t: FiniteSet[V], part: V -> Bool
) {
    is_bipartition(g, t, part) implies is_bipartition(induced_subgraph(g, s.underlying_set), t, part)
} by {
    if is_bipartition(g, t, part) {
        forall(x: V, y: V) {
            if t.contains(x) and t.contains(y) and induced_subgraph(g, s.underlying_set).adj(x, y) {
                finite_induced_adj_imp(g, s, x, y)
                g.adj(x, y)
                is_bipartition_apply(g, t, part, x, y)
                part(x) != part(y)
            }
            t.contains(x) and t.contains(y) and induced_subgraph(g, s.underlying_set).adj(x, y) implies part(x) != part(y)
        }
        is_bipartition_intro(induced_subgraph(g, s.underlying_set), t, part)
        is_bipartition(induced_subgraph(g, s.underlying_set), t, part)
    }
}

/// A vertex cover of a graph covers every induced subgraph.
theorem is_vertex_cover_induced[V](
    g: SimpleGraph[V], s: FiniteSet[V], t: FiniteSet[V], c: FiniteSet[V]
) {
    is_vertex_cover(g, t, c) implies is_vertex_cover(induced_subgraph(g, s.underlying_set), t, c)
} by {
    if is_vertex_cover(g, t, c) {
        forall(x: V, y: V) {
            if t.contains(x) and t.contains(y) and induced_subgraph(g, s.underlying_set).adj(x, y) {
                finite_induced_adj_imp(g, s, x, y)
                g.adj(x, y)
                is_vertex_cover_apply(g, t, c, x, y)
                c.contains(x) or c.contains(y)
            }
            t.contains(x) and t.contains(y) and induced_subgraph(g, s.underlying_set).adj(x, y) implies c.contains(x) or c.contains(y)
        }
        is_vertex_cover_intro(induced_subgraph(g, s.underlying_set), t, c)
        is_vertex_cover(induced_subgraph(g, s.underlying_set), t, c)
    }
}
