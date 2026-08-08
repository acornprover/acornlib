from nat import Nat
from pair import Pair, pair_first, pair_second
from finite_set import FiniteSet, fs_union, fs_image, finite_set_empty_subset,
    finite_set_subset_contains, finite_set_subset_antisymm,
    finite_set_disjoint_union_cardinality_is, finite_set_union_contains_eq,
    finite_set_image_contains_witness, finite_set_maps_into_image,
    finite_set_witness_predicate, finite_set_witness_intro,
    choose_from_finite_set_or_default, choose_from_finite_set_or_default_contains,
    choose_from_finite_set_or_default_property
from data.finite.finite_set_card import fs_card, fs_card_empty,
    fs_card_eq_of_cardinality_is, fs_card_cardinality_is
from data.finite.finite_set_card_compare import fs_card_mono
from data.finite.finite_set_subset_intro import fs_subset_eq_intro, fs_subset_union_of_contains
from data.finite.finite_fiber_partition import finite_fiber_partition_injective_fibers_cardinality_one,
    finite_fiber_partition_singleton_fibers_cardinality_eq_image
from data.nat.nat_bounded_max import is_max, has_max, is_max_apply, is_max_is_upper_bound,
    is_upper_bound_of, is_upper_bound_of_intro
from data.basic.logic import false_implies
from graph.simple_graph import SimpleGraph, simple_graph_adj_symmetric
from graph.simple_graph_edges import directed_edges, directed_edges_ne,
    directed_edges_first, directed_edges_second, directed_edges_adj,
    directed_edges_contains_pair, directed_edges_contains_eq
from graph.simple_graph_edge_fibers import directed_edges_from_set, directed_edges_from_set_contains_eq
from graph.simple_graph_bipartite import is_bipartition
from graph.simple_graph_bipartite_sides import bipartite_side

numerals Nat

/// True if the two pairs share at least one vertex.
define pairs_share_vertex[V](p: Pair[V, V], q: Pair[V, V]) -> Bool {
    p.first = q.first or p.first = q.second or p.second = q.first or p.second = q.second
}

/// A matching in `g` within the finite vertex set `s`: a set of edge pairs in which
/// no two distinct pairs share a vertex.
///
/// Edges are represented by directed pairs, as elsewhere in this library (`directed_edges`).
/// A matching contains at most one orientation of each edge, so the cardinality of the
/// pair set is the number of matching edges.
define is_matching[V](g: SimpleGraph[V], s: FiniteSet[V], m: FiniteSet[Pair[V, V]]) -> Bool {
    m.subset_eq(directed_edges(g, s)) and forall(p: Pair[V, V], q: Pair[V, V]) {
        m.contains(p) and m.contains(q) and pairs_share_vertex(p, q) implies p = q
    }
}

/// The size of a matching: the number of its edge pairs.
define matching_size[V](g: SimpleGraph[V], s: FiniteSet[V], m: FiniteSet[Pair[V, V]]) -> Nat {
    fs_card(m)
}

/// The vertices covered by a matching: the endpoints of its edge pairs.
define matching_vertices[V](m: FiniteSet[Pair[V, V]]) -> FiniteSet[V] {
    fs_union(fs_image(m, pair_first[V, V]), fs_image(m, pair_second[V, V]))
}

/// A perfect matching: a matching that covers every vertex of the ambient set.
define is_perfect_matching[V](g: SimpleGraph[V], s: FiniteSet[V], m: FiniteSet[Pair[V, V]]) -> Bool {
    is_matching(g, s, m) and forall(v: V) {
        s.contains(v) implies exists(p: Pair[V, V]) {
            m.contains(p) and (p.first = v or p.second = v)
        }
    }
}

/// A matching is contained in the directed edges of the ambient set.
theorem is_matching_apply_edge[V](g: SimpleGraph[V], s: FiniteSet[V], m: FiniteSet[Pair[V, V]], p: Pair[V, V]) {
    is_matching(g, s, m) and m.contains(p) implies directed_edges(g, s).contains(p)
} by {
    if is_matching(g, s, m) and m.contains(p) {
        is_matching(g, s, m) = (m.subset_eq(directed_edges(g, s)) and forall(q: Pair[V, V], r: Pair[V, V]) {
            m.contains(q) and m.contains(r) and pairs_share_vertex(q, r) implies q = r
        })
        m.subset_eq(directed_edges(g, s))
        finite_set_subset_contains(m, directed_edges(g, s), p)
        directed_edges(g, s).contains(p)
    }
}

/// Distinct matching pairs never share a vertex.
theorem is_matching_apply_disjoint[V](g: SimpleGraph[V], s: FiniteSet[V], m: FiniteSet[Pair[V, V]], p: Pair[V, V], q: Pair[V, V]) {
    is_matching(g, s, m) and m.contains(p) and m.contains(q) and pairs_share_vertex(p, q) implies p = q
} by {
    if is_matching(g, s, m) and m.contains(p) and m.contains(q) and pairs_share_vertex(p, q) {
        is_matching(g, s, m) = (m.subset_eq(directed_edges(g, s)) and forall(a: Pair[V, V], b: Pair[V, V]) {
            m.contains(a) and m.contains(b) and pairs_share_vertex(a, b) implies a = b
        })
        m.contains(p) and m.contains(q) and pairs_share_vertex(p, q)
        p = q
    }
}

/// Edge pairs contained in the directed edges with no shared vertices form a matching.
theorem is_matching_intro[V](g: SimpleGraph[V], s: FiniteSet[V], m: FiniteSet[Pair[V, V]]) {
    m.subset_eq(directed_edges(g, s)) and forall(p: Pair[V, V], q: Pair[V, V]) {
        m.contains(p) and m.contains(q) and pairs_share_vertex(p, q) implies p = q
    } implies is_matching(g, s, m)
} by {
    if m.subset_eq(directed_edges(g, s)) and forall(p: Pair[V, V], q: Pair[V, V]) {
        m.contains(p) and m.contains(q) and pairs_share_vertex(p, q) implies p = q
    } {
        is_matching(g, s, m) = (m.subset_eq(directed_edges(g, s)) and forall(a: Pair[V, V], b: Pair[V, V]) {
            m.contains(a) and m.contains(b) and pairs_share_vertex(a, b) implies a = b
        })
        is_matching(g, s, m)
    }
}

/// The components of a matching pair are distinct.
theorem matching_pair_ne[V](g: SimpleGraph[V], s: FiniteSet[V], m: FiniteSet[Pair[V, V]], p: Pair[V, V]) {
    is_matching(g, s, m) and m.contains(p) implies p.first != p.second
} by {
    if is_matching(g, s, m) and m.contains(p) {
        is_matching_apply_edge(g, s, m, p)
        directed_edges(g, s).contains(p)
        directed_edges_ne(g, s, p)
        p.first != p.second
    }
}

/// The components of a matching pair lie in the ambient set.
theorem matching_pair_components_in[V](g: SimpleGraph[V], s: FiniteSet[V], m: FiniteSet[Pair[V, V]], p: Pair[V, V]) {
    is_matching(g, s, m) and m.contains(p) implies s.contains(p.first) and s.contains(p.second)
} by {
    if is_matching(g, s, m) and m.contains(p) {
        is_matching_apply_edge(g, s, m, p)
        directed_edges(g, s).contains(p)
        directed_edges_first(g, s, p)
        directed_edges_second(g, s, p)
        s.contains(p.first) and s.contains(p.second)
    }
}

/// The first components of matching pairs are all distinct.
theorem matching_first_locally_injective[V](g: SimpleGraph[V], s: FiniteSet[V], m: FiniteSet[Pair[V, V]], p: Pair[V, V], q: Pair[V, V]) {
    is_matching(g, s, m) and m.contains(p) and m.contains(q) and p.first = q.first implies p = q
} by {
    if is_matching(g, s, m) and m.contains(p) and m.contains(q) and p.first = q.first {
        pairs_share_vertex(p, q) = (p.first = q.first or p.first = q.second or p.second = q.first or p.second = q.second)
        p.first = q.first
        p.first = q.first or p.first = q.second or p.second = q.first or p.second = q.second
        pairs_share_vertex(p, q)
        is_matching_apply_disjoint(g, s, m, p, q)
        p = q
    }
}

/// The second components of matching pairs are all distinct.
theorem matching_second_locally_injective[V](g: SimpleGraph[V], s: FiniteSet[V], m: FiniteSet[Pair[V, V]], p: Pair[V, V], q: Pair[V, V]) {
    is_matching(g, s, m) and m.contains(p) and m.contains(q) and p.second = q.second implies p = q
} by {
    if is_matching(g, s, m) and m.contains(p) and m.contains(q) and p.second = q.second {
        pairs_share_vertex(p, q) = (p.first = q.first or p.first = q.second or p.second = q.first or p.second = q.second)
        p.second = q.second
        p.first = q.first or p.first = q.second or p.second = q.first or p.second = q.second
        pairs_share_vertex(p, q)
        is_matching_apply_disjoint(g, s, m, p, q)
        p = q
    }
}

/// A map that is injective on the members of a finite set preserves its cardinality.
///
/// Built on the fiber partition: injectivity makes every canonical fiber a singleton,
/// and singleton fibers make the source and image counts agree.
theorem fs_card_image_eq_of_locally_injective[T, U](s: FiniteSet[T], f: T -> U) {
    (forall(x: T, y: T) {
        s.contains(x) and s.contains(y) and f(x) = f(y) implies x = y
    }) implies fs_card(fs_image(s, f)) = fs_card(s)
} by {
    if forall(x: T, y: T) {
        s.contains(x) and s.contains(y) and f(x) = f(y) implies x = y
    } {
        finite_fiber_partition_injective_fibers_cardinality_one(s, f)
        fs_card_cardinality_is(fs_image(s, f))
        fs_image(s, f).cardinality_is(fs_card(fs_image(s, f)))
        finite_fiber_partition_singleton_fibers_cardinality_eq_image(s, f, fs_card(fs_image(s, f)))
        s.cardinality_is(fs_card(fs_image(s, f)))
        fs_card_eq_of_cardinality_is(s, fs_card(fs_image(s, f)))
        fs_card(s) = fs_card(fs_image(s, f))
    }
}

/// The first-endpoint image of a matching has the same size as the matching.
theorem matching_first_image_card[V](g: SimpleGraph[V], s: FiniteSet[V], m: FiniteSet[Pair[V, V]]) {
    is_matching(g, s, m) implies fs_card(fs_image(m, pair_first[V, V])) = fs_card(m)
} by {
    if is_matching(g, s, m) {
        forall(p: Pair[V, V], q: Pair[V, V]) {
            if m.contains(p) and m.contains(q) and pair_first(p) = pair_first(q) {
                pair_first(p) = p.first
                pair_first(q) = q.first
                p.first = q.first
                matching_first_locally_injective(g, s, m, p, q)
                p = q
            }
            m.contains(p) and m.contains(q) and pair_first(p) = pair_first(q) implies p = q
        }
        fs_card_image_eq_of_locally_injective(m, pair_first[V, V])
        fs_card(fs_image(m, pair_first[V, V])) = fs_card(m)
    }
}

/// The second-endpoint image of a matching has the same size as the matching.
theorem matching_second_image_card[V](g: SimpleGraph[V], s: FiniteSet[V], m: FiniteSet[Pair[V, V]]) {
    is_matching(g, s, m) implies fs_card(fs_image(m, pair_second[V, V])) = fs_card(m)
} by {
    if is_matching(g, s, m) {
        forall(p: Pair[V, V], q: Pair[V, V]) {
            if m.contains(p) and m.contains(q) and pair_second(p) = pair_second(q) {
                pair_second(p) = p.second
                pair_second(q) = q.second
                p.second = q.second
                matching_second_locally_injective(g, s, m, p, q)
                p = q
            }
            m.contains(p) and m.contains(q) and pair_second(p) = pair_second(q) implies p = q
        }
        fs_card_image_eq_of_locally_injective(m, pair_second[V, V])
        fs_card(fs_image(m, pair_second[V, V])) = fs_card(m)
    }
}

/// The first- and second-endpoint images of a matching are disjoint.
///
/// A vertex shared between a first component and a second component would lie on two
/// distinct matching pairs, which the matching condition forbids.
theorem matching_images_disjoint[V](g: SimpleGraph[V], s: FiniteSet[V], m: FiniteSet[Pair[V, V]]) {
    is_matching(g, s, m) implies fs_image(m, pair_first[V, V]).is_disjoint(fs_image(m, pair_second[V, V]))
} by {
    if is_matching(g, s, m) {
        forall(v: V) {
            if fs_image(m, pair_first[V, V]).contains(v) and fs_image(m, pair_second[V, V]).contains(v) {
                finite_set_image_contains_witness(m, pair_first[V, V], v)
                let p: Pair[V, V] satisfy {
                    m.contains(p) and v = pair_first(p)
                }
                finite_set_image_contains_witness(m, pair_second[V, V], v)
                let q: Pair[V, V] satisfy {
                    m.contains(q) and v = pair_second(q)
                }
                pair_first(p) = p.first
                pair_second(q) = q.second
                v = p.first
                v = q.second
                p.first = q.second
                pairs_share_vertex(p, q) = (p.first = q.first or p.first = q.second or p.second = q.first or p.second = q.second)
                p.first = q.second
                p.first = q.first or p.first = q.second or p.second = q.first or p.second = q.second
                pairs_share_vertex(p, q)
                is_matching_apply_disjoint(g, s, m, p, q)
                p = q
                p.first = p.second
                matching_pair_ne(g, s, m, p)
                p.first != p.second
                false
            }
            not (fs_image(m, pair_first[V, V]).contains(v) and fs_image(m, pair_second[V, V]).contains(v))
            fs_image(m, pair_first[V, V]).contains(v) = fs_image(m, pair_first[V, V]).underlying_set.contains(v)
            fs_image(m, pair_second[V, V]).contains(v) = fs_image(m, pair_second[V, V]).underlying_set.contains(v)
            not (fs_image(m, pair_first[V, V]).underlying_set.contains(v) and fs_image(m, pair_second[V, V]).underlying_set.contains(v))
        }
        fs_image(m, pair_first[V, V]).underlying_set.is_disjoint(fs_image(m, pair_second[V, V]).underlying_set) = forall(v: V) {
            not (fs_image(m, pair_first[V, V]).underlying_set.contains(v) and fs_image(m, pair_second[V, V]).underlying_set.contains(v))
        }
        fs_image(m, pair_first[V, V]).underlying_set.is_disjoint(fs_image(m, pair_second[V, V]).underlying_set)
        fs_image(m, pair_first[V, V]).is_disjoint(fs_image(m, pair_second[V, V]))
    }
}

/// The endpoint set of a matching has twice as many vertices as the matching has edges.
theorem matching_vertices_card[V](g: SimpleGraph[V], s: FiniteSet[V], m: FiniteSet[Pair[V, V]]) {
    is_matching(g, s, m) implies fs_card(matching_vertices(m)) = fs_card(m) + fs_card(m)
} by {
    if is_matching(g, s, m) {
        matching_first_image_card(g, s, m)
        fs_card(fs_image(m, pair_first[V, V])) = fs_card(m)
        matching_second_image_card(g, s, m)
        fs_card(fs_image(m, pair_second[V, V])) = fs_card(m)
        matching_images_disjoint(g, s, m)
        fs_image(m, pair_first[V, V]).is_disjoint(fs_image(m, pair_second[V, V]))
        fs_card_cardinality_is(fs_image(m, pair_first[V, V]))
        fs_image(m, pair_first[V, V]).cardinality_is(fs_card(fs_image(m, pair_first[V, V])))
        fs_card_cardinality_is(fs_image(m, pair_second[V, V]))
        fs_image(m, pair_second[V, V]).cardinality_is(fs_card(fs_image(m, pair_second[V, V])))
        finite_set_disjoint_union_cardinality_is(fs_image(m, pair_first[V, V]), fs_image(m, pair_second[V, V]),
            fs_card(fs_image(m, pair_first[V, V])), fs_card(fs_image(m, pair_second[V, V])))
        fs_union(fs_image(m, pair_first[V, V]), fs_image(m, pair_second[V, V])).cardinality_is(
            fs_card(fs_image(m, pair_first[V, V])) + fs_card(fs_image(m, pair_second[V, V])))
        matching_vertices(m) = fs_union(fs_image(m, pair_first[V, V]), fs_image(m, pair_second[V, V]))
        matching_vertices(m).cardinality_is(fs_card(m) + fs_card(m))
        fs_card_eq_of_cardinality_is(matching_vertices(m), fs_card(m) + fs_card(m))
        fs_card(matching_vertices(m)) = fs_card(m) + fs_card(m)
    }
}

/// The first endpoints of a matching lie in the ambient vertex set.
theorem matching_first_image_subset_ambient[V](g: SimpleGraph[V], s: FiniteSet[V], m: FiniteSet[Pair[V, V]]) {
    is_matching(g, s, m) implies fs_image(m, pair_first[V, V]).subset_eq(s)
} by {
    if is_matching(g, s, m) {
        forall(v: V) {
            if fs_image(m, pair_first[V, V]).contains(v) {
                finite_set_image_contains_witness(m, pair_first[V, V], v)
                let p: Pair[V, V] satisfy {
                    m.contains(p) and v = pair_first(p)
                }
                pair_first(p) = p.first
                v = p.first
                is_matching_apply_edge(g, s, m, p)
                directed_edges(g, s).contains(p)
                directed_edges_first(g, s, p)
                s.contains(p.first)
                s.contains(v)
            }
            fs_image(m, pair_first[V, V]).contains(v) implies s.contains(v)
        }
        fs_subset_eq_intro(fs_image(m, pair_first[V, V]), s)
        fs_image(m, pair_first[V, V]).subset_eq(s)
    }
}

/// The second endpoints of a matching lie in the ambient vertex set.
theorem matching_second_image_subset_ambient[V](g: SimpleGraph[V], s: FiniteSet[V], m: FiniteSet[Pair[V, V]]) {
    is_matching(g, s, m) implies fs_image(m, pair_second[V, V]).subset_eq(s)
} by {
    if is_matching(g, s, m) {
        forall(v: V) {
            if fs_image(m, pair_second[V, V]).contains(v) {
                finite_set_image_contains_witness(m, pair_second[V, V], v)
                let p: Pair[V, V] satisfy {
                    m.contains(p) and v = pair_second(p)
                }
                pair_second(p) = p.second
                v = p.second
                is_matching_apply_edge(g, s, m, p)
                directed_edges(g, s).contains(p)
                directed_edges_second(g, s, p)
                s.contains(p.second)
                s.contains(v)
            }
            fs_image(m, pair_second[V, V]).contains(v) implies s.contains(v)
        }
        fs_subset_eq_intro(fs_image(m, pair_second[V, V]), s)
        fs_image(m, pair_second[V, V]).subset_eq(s)
    }
}

/// The endpoints of a matching lie in the ambient vertex set.
theorem matching_vertices_subset_ambient[V](g: SimpleGraph[V], s: FiniteSet[V], m: FiniteSet[Pair[V, V]]) {
    is_matching(g, s, m) implies matching_vertices(m).subset_eq(s)
} by {
    if is_matching(g, s, m) {
        matching_first_image_subset_ambient(g, s, m)
        fs_image(m, pair_first[V, V]).subset_eq(s)
        matching_second_image_subset_ambient(g, s, m)
        fs_image(m, pair_second[V, V]).subset_eq(s)
        fs_subset_union_of_contains(fs_image(m, pair_first[V, V]), fs_image(m, pair_second[V, V]), s)
        fs_union(fs_image(m, pair_first[V, V]), fs_image(m, pair_second[V, V])).subset_eq(s)
        matching_vertices(m) = fs_union(fs_image(m, pair_first[V, V]), fs_image(m, pair_second[V, V]))
        matching_vertices(m).subset_eq(s)
    }
}

/// Twice the size of a matching is at most the number of vertices.
///
/// Each matching edge uses two distinct vertices, and distinct edges use disjoint
/// vertices, so the endpoints form a subset of `s` of cardinality twice the size.
theorem matching_size_le_half_vertices[V](g: SimpleGraph[V], s: FiniteSet[V], m: FiniteSet[Pair[V, V]]) {
    is_matching(g, s, m) implies fs_card(m) + fs_card(m) <= fs_card(s)
} by {
    if is_matching(g, s, m) {
        matching_vertices_card(g, s, m)
        fs_card(matching_vertices(m)) = fs_card(m) + fs_card(m)
        matching_vertices_subset_ambient(g, s, m)
        matching_vertices(m).subset_eq(s)
        fs_card_mono(matching_vertices(m), s)
        fs_card(matching_vertices(m)) <= fs_card(s)
        fs_card(m) + fs_card(m) <= fs_card(s)
    }
}

/// The size of a matching is at most the number of vertices.
theorem matching_size_le_vertices[V](g: SimpleGraph[V], s: FiniteSet[V], m: FiniteSet[Pair[V, V]]) {
    is_matching(g, s, m) implies fs_card(m) <= fs_card(s)
} by {
    if is_matching(g, s, m) {
        matching_first_image_card(g, s, m)
        fs_card(fs_image(m, pair_first[V, V])) = fs_card(m)
        matching_first_image_subset_ambient(g, s, m)
        fs_image(m, pair_first[V, V]).subset_eq(s)
        fs_card_mono(fs_image(m, pair_first[V, V]), s)
        fs_card(fs_image(m, pair_first[V, V])) <= fs_card(s)
        fs_card(m) <= fs_card(s)
    }
}

/// True of the sizes achieved by matchings of `g` in `s`.
define matching_size_pred[V](g: SimpleGraph[V], s: FiniteSet[V]) -> (Nat -> Bool) {
    function(n: Nat) {
        exists(m: FiniteSet[Pair[V, V]]) {
            is_matching(g, s, m) and fs_card(m) = n
        }
    }
}

/// A matching achieves its own size.
theorem matching_size_pred_intro[V](g: SimpleGraph[V], s: FiniteSet[V], m: FiniteSet[Pair[V, V]]) {
    is_matching(g, s, m) implies matching_size_pred(g, s)(fs_card(m))
} by {
    if is_matching(g, s, m) {
        matching_size_pred(g, s)(fs_card(m)) = exists(r: FiniteSet[Pair[V, V]]) {
            is_matching(g, s, r) and fs_card(r) = fs_card(m)
        }
        exists(r: FiniteSet[Pair[V, V]]) {
            is_matching(g, s, r) and fs_card(r) = fs_card(m)
        }
        matching_size_pred(g, s)(fs_card(m))
    }
}

/// The empty set of pairs is a matching.
theorem empty_set_is_matching[V](g: SimpleGraph[V], s: FiniteSet[V]) {
    is_matching(g, s, FiniteSet.empty[Pair[V, V]])
} by {
    finite_set_empty_subset(directed_edges(g, s))
    FiniteSet.empty[Pair[V, V]].subset_eq(directed_edges(g, s))
    forall(p: Pair[V, V], q: Pair[V, V]) {
        not FiniteSet.empty[Pair[V, V]].contains(p)
        (FiniteSet.empty[Pair[V, V]].contains(p) and FiniteSet.empty[Pair[V, V]].contains(q) and pairs_share_vertex(p, q)) = false
        false_implies(p = q)
        (false implies p = q) = true
        ((FiniteSet.empty[Pair[V, V]].contains(p) and FiniteSet.empty[Pair[V, V]].contains(q) and pairs_share_vertex(p, q)) implies p = q) = true
        (FiniteSet.empty[Pair[V, V]].contains(p) and FiniteSet.empty[Pair[V, V]].contains(q) and pairs_share_vertex(p, q)) implies p = q
    }
    is_matching_intro(g, s, FiniteSet.empty[Pair[V, V]])
    is_matching(g, s, FiniteSet.empty[Pair[V, V]])
}

/// Zero is an achieved matching size, witnessed by the empty matching.
theorem matching_size_pred_zero[V](g: SimpleGraph[V], s: FiniteSet[V]) {
    matching_size_pred(g, s)(Nat.0)
} by {
    empty_set_is_matching(g, s)
    is_matching(g, s, FiniteSet.empty[Pair[V, V]])
    matching_size_pred_intro(g, s, FiniteSet.empty[Pair[V, V]])
    matching_size_pred(g, s)(fs_card(FiniteSet.empty[Pair[V, V]]))
    fs_card_empty[Pair[V, V]]
    fs_card(FiniteSet.empty[Pair[V, V]]) = Nat.0
    matching_size_pred(g, s)(Nat.0)
}

/// The number of vertices bounds every matching size.
theorem matching_size_pred_bounded[V](g: SimpleGraph[V], s: FiniteSet[V]) {
    is_upper_bound_of(matching_size_pred(g, s), fs_card(s))
} by {
    forall(n: Nat) {
        if matching_size_pred(g, s)(n) {
            matching_size_pred(g, s)(n) = exists(m: FiniteSet[Pair[V, V]]) {
                is_matching(g, s, m) and fs_card(m) = n
            }
            exists(m: FiniteSet[Pair[V, V]]) {
                is_matching(g, s, m) and fs_card(m) = n
            }
            let (m: FiniteSet[Pair[V, V]]) satisfy {
                is_matching(g, s, m) and fs_card(m) = n
            }
            matching_size_le_vertices(g, s, m)
            fs_card(m) <= fs_card(s)
            n <= fs_card(s)
        }
        (matching_size_pred(g, s)(n) implies n <= fs_card(s))
    }
    is_upper_bound_of_intro(matching_size_pred(g, s), fs_card(s))
    is_upper_bound_of(matching_size_pred(g, s), fs_card(s))
}

/// The size of a largest matching of `g` in `s`.
///
/// The matching number. Well defined because the empty matching exists and every
/// matching size is bounded by the number of vertices.
let maximum_matching_size[V](g: SimpleGraph[V], s: FiniteSet[V]) -> result: Nat satisfy {
    is_max(matching_size_pred(g, s), result)
} by {
    matching_size_pred_zero(g, s)
    matching_size_pred_bounded(g, s)
    has_max(matching_size_pred(g, s), Nat.0, fs_card(s))
}

/// Some matching attains the maximum matching size.
theorem maximum_matching_size_attained[V](g: SimpleGraph[V], s: FiniteSet[V]) {
    matching_size_pred(g, s)(maximum_matching_size(g, s))
} by {
    is_max(matching_size_pred(g, s), maximum_matching_size(g, s))
    is_max_apply(matching_size_pred(g, s), maximum_matching_size(g, s))
}

/// No matching is larger than the maximum matching size.
theorem maximum_matching_size_is_greatest[V](g: SimpleGraph[V], s: FiniteSet[V], m: FiniteSet[Pair[V, V]]) {
    is_matching(g, s, m) implies fs_card(m) <= maximum_matching_size(g, s)
} by {
    if is_matching(g, s, m) {
        matching_size_pred_intro(g, s, m)
        matching_size_pred(g, s)(fs_card(m))
        is_max(matching_size_pred(g, s), maximum_matching_size(g, s))
        is_max_is_upper_bound(matching_size_pred(g, s), maximum_matching_size(g, s), fs_card(m))
        fs_card(m) <= maximum_matching_size(g, s)
    }
}

/// A matching of the largest size exists.
theorem maximum_matching_exists[V](g: SimpleGraph[V], s: FiniteSet[V]) {
    exists(m: FiniteSet[Pair[V, V]]) {
        is_matching(g, s, m) and fs_card(m) = maximum_matching_size(g, s)
    }
} by {
    maximum_matching_size_attained(g, s)
    matching_size_pred(g, s)(maximum_matching_size(g, s))
    matching_size_pred(g, s)(maximum_matching_size(g, s)) = exists(m: FiniteSet[Pair[V, V]]) {
        is_matching(g, s, m) and fs_card(m) = maximum_matching_size(g, s)
    }
    exists(m: FiniteSet[Pair[V, V]]) {
        is_matching(g, s, m) and fs_card(m) = maximum_matching_size(g, s)
    }
}

/// A perfect matching covers the vertex it is applied to.
theorem is_perfect_matching_apply_cover[V](g: SimpleGraph[V], s: FiniteSet[V], m: FiniteSet[Pair[V, V]], v: V) {
    is_perfect_matching(g, s, m) and s.contains(v) implies exists(p: Pair[V, V]) {
        m.contains(p) and (p.first = v or p.second = v)
    }
} by {
    if is_perfect_matching(g, s, m) and s.contains(v) {
        is_perfect_matching(g, s, m) = (is_matching(g, s, m) and forall(u: V) {
            s.contains(u) implies exists(p: Pair[V, V]) {
                m.contains(p) and (p.first = u or p.second = u)
            }
        })
        forall(u: V) {
            s.contains(u) implies exists(p: Pair[V, V]) {
                m.contains(p) and (p.first = u or p.second = u)
            }
        }
        exists(p: Pair[V, V]) {
            m.contains(p) and (p.first = v or p.second = v)
        }
    }
}

/// A perfect matching is a matching.
theorem perfect_matching_is_matching[V](g: SimpleGraph[V], s: FiniteSet[V], m: FiniteSet[Pair[V, V]]) {
    is_perfect_matching(g, s, m) implies is_matching(g, s, m)
} by {
    if is_perfect_matching(g, s, m) {
        is_perfect_matching(g, s, m) = (is_matching(g, s, m) and forall(u: V) {
            s.contains(u) implies exists(p: Pair[V, V]) {
                m.contains(p) and (p.first = u or p.second = u)
            }
        })
        is_matching(g, s, m)
    }
}

/// A matching that covers every vertex is perfect.
theorem is_perfect_matching_intro[V](g: SimpleGraph[V], s: FiniteSet[V], m: FiniteSet[Pair[V, V]]) {
    is_matching(g, s, m) and (forall(v: V) {
        s.contains(v) implies exists(p: Pair[V, V]) {
            m.contains(p) and (p.first = v or p.second = v)
        }
    }) implies is_perfect_matching(g, s, m)
} by {
    if is_matching(g, s, m) and forall(v: V) {
        s.contains(v) implies exists(p: Pair[V, V]) {
            m.contains(p) and (p.first = v or p.second = v)
        }
    } {
        is_perfect_matching(g, s, m) = (is_matching(g, s, m) and forall(u: V) {
            s.contains(u) implies exists(p: Pair[V, V]) {
                m.contains(p) and (p.first = u or p.second = u)
            }
        })
        is_perfect_matching(g, s, m)
    }
}

/// The endpoint set of a perfect matching is the whole vertex set.
theorem perfect_matching_vertices_cover[V](g: SimpleGraph[V], s: FiniteSet[V], m: FiniteSet[Pair[V, V]]) {
    is_perfect_matching(g, s, m) implies matching_vertices(m) = s
} by {
    if is_perfect_matching(g, s, m) {
        perfect_matching_is_matching(g, s, m)
        is_matching(g, s, m)
        matching_vertices_subset_ambient(g, s, m)
        matching_vertices(m).subset_eq(s)
        forall(v: V) {
            if s.contains(v) {
                is_perfect_matching_apply_cover(g, s, m, v)
                let p: Pair[V, V] satisfy {
                    m.contains(p) and (p.first = v or p.second = v)
                }
                if p.first = v {
                    finite_set_maps_into_image(m, pair_first[V, V], p)
                    fs_image(m, pair_first[V, V]).contains(pair_first(p))
                    pair_first(p) = p.first
                    fs_image(m, pair_first[V, V]).contains(v)
                    finite_set_union_contains_eq(fs_image(m, pair_first[V, V]), fs_image(m, pair_second[V, V]), v)
                    fs_union(fs_image(m, pair_first[V, V]), fs_image(m, pair_second[V, V])).contains(v)
                    matching_vertices(m) = fs_union(fs_image(m, pair_first[V, V]), fs_image(m, pair_second[V, V]))
                    matching_vertices(m).contains(v)
                }
                if not (p.first = v) {
                    p.second = v
                    finite_set_maps_into_image(m, pair_second[V, V], p)
                    fs_image(m, pair_second[V, V]).contains(pair_second(p))
                    pair_second(p) = p.second
                    fs_image(m, pair_second[V, V]).contains(v)
                    finite_set_union_contains_eq(fs_image(m, pair_first[V, V]), fs_image(m, pair_second[V, V]), v)
                    fs_union(fs_image(m, pair_first[V, V]), fs_image(m, pair_second[V, V])).contains(v)
                    matching_vertices(m) = fs_union(fs_image(m, pair_first[V, V]), fs_image(m, pair_second[V, V]))
                    matching_vertices(m).contains(v)
                }
                matching_vertices(m).contains(v)
            }
            s.contains(v) implies matching_vertices(m).contains(v)
        }
        fs_subset_eq_intro(s, matching_vertices(m))
        s.subset_eq(matching_vertices(m))
        finite_set_subset_antisymm(matching_vertices(m), s)
        matching_vertices(m) = s
    }
}

/// A perfect matching has twice as many edges as the vertex set has vertices.
theorem perfect_matching_size[V](g: SimpleGraph[V], s: FiniteSet[V], m: FiniteSet[Pair[V, V]]) {
    is_perfect_matching(g, s, m) implies fs_card(s) = fs_card(m) + fs_card(m)
} by {
    if is_perfect_matching(g, s, m) {
        perfect_matching_is_matching(g, s, m)
        is_matching(g, s, m)
        matching_vertices_card(g, s, m)
        fs_card(matching_vertices(m)) = fs_card(m) + fs_card(m)
        perfect_matching_vertices_cover(g, s, m)
        matching_vertices(m) = s
        fs_card(matching_vertices(m)) = fs_card(s)
        fs_card(s) = fs_card(m) + fs_card(m)
    }
}

/// The neighbors (within `s`) of the vertices of `t`: the second endpoints of the
/// directed edges leaving `t`.
define neighborhood_of_set[V](g: SimpleGraph[V], s: FiniteSet[V], t: FiniteSet[V]) -> FiniteSet[V] {
    fs_image(directed_edges_from_set(g, s, t), pair_second[V, V])
}

/// A neighbor of a vertex of `t` lies in the neighborhood of `t`.
theorem neighborhood_of_set_contains_of_adj[V](g: SimpleGraph[V], s: FiniteSet[V], t: FiniteSet[V], x: V, y: V) {
    t.contains(x) and s.contains(x) and s.contains(y) and g.adj(x, y)
        implies neighborhood_of_set(g, s, t).contains(y)
} by {
    if t.contains(x) and s.contains(x) and s.contains(y) and g.adj(x, y) {
        directed_edges_contains_pair(g, s, x, y)
        directed_edges(g, s).contains(Pair.new(x, y))
        directed_edges_from_set_contains_eq(g, s, t, Pair.new(x, y))
        directed_edges_from_set(g, s, t).contains(Pair.new(x, y)) = (directed_edges(g, s).contains(Pair.new(x, y)) and t.contains(Pair.new(x, y).first))
        Pair.new(x, y).first = x
        directed_edges_from_set(g, s, t).contains(Pair.new(x, y))
        pair_second(Pair.new(x, y)) = Pair.new(x, y).second
        Pair.new(x, y).second = y
        pair_second(Pair.new(x, y)) = y
        finite_set_maps_into_image(directed_edges_from_set(g, s, t), pair_second[V, V], Pair.new(x, y))
        fs_image(directed_edges_from_set(g, s, t), pair_second[V, V]).contains(pair_second(Pair.new(x, y)))
        fs_image(directed_edges_from_set(g, s, t), pair_second[V, V]).contains(y)
        neighborhood_of_set(g, s, t) = fs_image(directed_edges_from_set(g, s, t), pair_second[V, V])
        neighborhood_of_set(g, s, t).contains(y)
    }
}

/// Membership in the neighborhood of `t` is witnessed by an adjacent vertex of `t`.
theorem neighborhood_of_set_contains_imp_adj[V](g: SimpleGraph[V], s: FiniteSet[V], t: FiniteSet[V], y: V) {
    neighborhood_of_set(g, s, t).contains(y) implies exists(x: V) {
        t.contains(x) and s.contains(x) and s.contains(y) and g.adj(x, y)
    }
} by {
    if neighborhood_of_set(g, s, t).contains(y) {
        neighborhood_of_set(g, s, t) = fs_image(directed_edges_from_set(g, s, t), pair_second[V, V])
        finite_set_image_contains_witness(directed_edges_from_set(g, s, t), pair_second[V, V], y)
        let p: Pair[V, V] satisfy {
            directed_edges_from_set(g, s, t).contains(p) and y = pair_second(p)
        }
        directed_edges_from_set_contains_eq(g, s, t, p)
        directed_edges_from_set(g, s, t).contains(p) = (directed_edges(g, s).contains(p) and t.contains(p.first))
        directed_edges(g, s).contains(p)
        t.contains(p.first)
        directed_edges_contains_eq(g, s, p)
        directed_edges(g, s).contains(p) = (s.contains(p.first) and s.contains(p.second) and g.adj(p.first, p.second))
        s.contains(p.first)
        s.contains(p.second)
        g.adj(p.first, p.second)
        pair_second(p) = p.second
        y = p.second
        s.contains(p.second) and s.contains(y)
        s.contains(y)
        g.adj(p.first, p.second) and y = p.second
        g.adj(p.first, y)
        exists(x: V) {
            t.contains(x) and s.contains(x) and s.contains(y) and g.adj(x, y)
        }
    }
}

/// True when `v` lies on a pair of the matching `m`.
define covers_vertex[V](m: FiniteSet[Pair[V, V]], v: V) -> Bool {
    exists(p: Pair[V, V]) {
        m.contains(p) and (p.first = v or p.second = v)
    }
}

/// A matching covers a vertex set when every vertex of the set lies on a matching pair.
define covers[V](m: FiniteSet[Pair[V, V]], t: FiniteSet[V]) -> Bool {
    forall(v: V) {
        t.contains(v) implies covers_vertex(m, v)
    }
}

/// The covering condition applied to one vertex.
theorem covers_apply[V](m: FiniteSet[Pair[V, V]], t: FiniteSet[V], v: V) {
    covers(m, t) and t.contains(v) implies exists(p: Pair[V, V]) {
        m.contains(p) and (p.first = v or p.second = v)
    }
} by {
    if covers(m, t) and t.contains(v) {
        covers(m, t) = forall(u: V) {
            t.contains(u) implies covers_vertex(m, u)
        }
        t.contains(v) implies covers_vertex(m, v)
        covers_vertex(m, v) = exists(p: Pair[V, V]) {
            m.contains(p) and (p.first = v or p.second = v)
        }
        exists(p: Pair[V, V]) {
            m.contains(p) and (p.first = v or p.second = v)
        }
    }
}

/// The covering condition, introduced pointwise.
theorem covers_intro[V](m: FiniteSet[Pair[V, V]], t: FiniteSet[V]) {
    (forall(v: V) {
        t.contains(v) implies covers_vertex(m, v)
    }) implies covers(m, t)
} by {
    if forall(v: V) {
        t.contains(v) implies covers_vertex(m, v)
    } {
        covers(m, t) = forall(u: V) {
            t.contains(u) implies covers_vertex(m, u)
        }
        covers(m, t)
    }
}

/// Covering passes from a vertex set to any subset.
theorem covers_of_subset[V](m: FiniteSet[Pair[V, V]], t: FiniteSet[V], side: FiniteSet[V]) {
    covers(m, side) and t.subset_eq(side) implies covers(m, t)
} by {
    if covers(m, side) and t.subset_eq(side) {
        forall(v: V) {
            if t.contains(v) {
                finite_set_subset_contains(t, side, v)
                side.contains(v)
                covers_apply(m, side, v)
                exists(p: Pair[V, V]) {
                    m.contains(p) and (p.first = v or p.second = v)
                }
                covers_vertex(m, v) = exists(p: Pair[V, V]) {
                    m.contains(p) and (p.first = v or p.second = v)
                }
                covers_vertex(m, v)
            }
            t.contains(v) implies covers_vertex(m, v)
        }
        covers_intro(m, t)
        covers(m, t)
    }
}

/// True of the matching pairs that cover the vertex `v`.
define covers_vertex_pred[V](m: FiniteSet[Pair[V, V]], v: V) -> (Pair[V, V] -> Bool) {
    function(p: Pair[V, V]) {
        m.contains(p) and (p.first = v or p.second = v)
    }
}

/// A vertex covered by a matching has a witness pair in the matching.
theorem covers_vertex_witness[V](m: FiniteSet[Pair[V, V]], t: FiniteSet[V], v: V) {
    covers(m, t) and t.contains(v) implies exists(p: Pair[V, V]) {
        finite_set_witness_predicate(m, covers_vertex_pred(m, v), p)
    }
} by {
    if covers(m, t) and t.contains(v) {
        covers_apply(m, t, v)
        let p: Pair[V, V] satisfy {
            m.contains(p) and (p.first = v or p.second = v)
        }
        covers_vertex_pred(m, v)(p) = (m.contains(p) and (p.first = v or p.second = v))
        finite_set_witness_intro(m, covers_vertex_pred(m, v), p)
        finite_set_witness_predicate(m, covers_vertex_pred(m, v), p)
        exists(w: Pair[V, V]) {
            finite_set_witness_predicate(m, covers_vertex_pred(m, v), w)
        }
    }
}

/// The pair chosen to cover `v`, or the degenerate pair `(v, v)` when none exists.
define covering_pair[V](m: FiniteSet[Pair[V, V]], v: V) -> Pair[V, V] {
    choose_from_finite_set_or_default(m, covers_vertex_pred(m, v), Pair.new(v, v))
}

/// The partner of `v` in the chosen covering pair of `m`, or `v` itself when no
/// covering pair exists.
define matching_partner[V](m: FiniteSet[Pair[V, V]], v: V) -> V {
    if covering_pair(m, v).first = v {
        covering_pair(m, v).second
    } else {
        covering_pair(m, v).first
    }
}

/// The chosen covering pair of a covered vertex lies in the matching.
theorem matching_partner_pair_contains[V](m: FiniteSet[Pair[V, V]], t: FiniteSet[V], v: V) {
    covers(m, t) and t.contains(v) implies m.contains(covering_pair(m, v))
} by {
    if covers(m, t) and t.contains(v) {
        covers_vertex_witness(m, t, v)
        choose_from_finite_set_or_default_contains(m, covers_vertex_pred(m, v), Pair.new(v, v))
        m.contains(covering_pair(m, v))
    }
}

/// The chosen covering pair of a covered vertex covers it.
theorem matching_partner_pair_covers[V](m: FiniteSet[Pair[V, V]], t: FiniteSet[V], v: V) {
    covers(m, t) and t.contains(v) implies covers_vertex_pred(m, v)(covering_pair(m, v))
} by {
    if covers(m, t) and t.contains(v) {
        covers_vertex_witness(m, t, v)
        choose_from_finite_set_or_default_property(m, covers_vertex_pred(m, v), Pair.new(v, v))
        covers_vertex_pred(m, v)(covering_pair(m, v))
    }
}

/// The partner of a covered vertex is not the vertex itself.
theorem matching_partner_ne[V](g: SimpleGraph[V], s: FiniteSet[V], m: FiniteSet[Pair[V, V]], t: FiniteSet[V], v: V) {
    is_matching(g, s, m) and covers(m, t) and t.contains(v) implies matching_partner(m, v) != v
} by {
    if is_matching(g, s, m) and covers(m, t) and t.contains(v) {
        matching_partner_pair_contains(m, t, v)
        m.contains(covering_pair(m, v))
        matching_partner_pair_covers(m, t, v)
        covers_vertex_pred(m, v)(covering_pair(m, v)) =
            (m.contains(covering_pair(m, v)) and (covering_pair(m, v).first = v or covering_pair(m, v).second = v))
        covering_pair(m, v).first = v or covering_pair(m, v).second = v
        if covering_pair(m, v).first = v {
            matching_partner(m, v) = covering_pair(m, v).second
            matching_pair_ne(g, s, m, covering_pair(m, v))
            covering_pair(m, v).first != covering_pair(m, v).second
            matching_partner(m, v) != v
        }
        if not (covering_pair(m, v).first = v) {
            matching_partner(m, v) = covering_pair(m, v).first
            covering_pair(m, v).second = v
            matching_pair_ne(g, s, m, covering_pair(m, v))
            covering_pair(m, v).first != covering_pair(m, v).second
            matching_partner(m, v) != v
        }
        matching_partner(m, v) != v
    }
}

/// The partner of a covered vertex is a neighbor of the covered set.
theorem matching_partner_in_neighborhood[V](g: SimpleGraph[V], s: FiniteSet[V], m: FiniteSet[Pair[V, V]], t: FiniteSet[V], v: V) {
    is_matching(g, s, m) and covers(m, t) and t.contains(v)
        implies neighborhood_of_set(g, s, t).contains(matching_partner(m, v))
} by {
    if is_matching(g, s, m) and covers(m, t) and t.contains(v) {
        matching_partner_pair_contains(m, t, v)
        m.contains(covering_pair(m, v))
        matching_partner_pair_covers(m, t, v)
        covers_vertex_pred(m, v)(covering_pair(m, v)) =
            (m.contains(covering_pair(m, v)) and (covering_pair(m, v).first = v or covering_pair(m, v).second = v))
        covering_pair(m, v).first = v or covering_pair(m, v).second = v
        if covering_pair(m, v).first = v {
            matching_partner(m, v) = covering_pair(m, v).second
            is_matching_apply_edge(g, s, m, covering_pair(m, v))
            directed_edges(g, s).contains(covering_pair(m, v))
            directed_edges_adj(g, s, covering_pair(m, v))
            g.adj(covering_pair(m, v).first, covering_pair(m, v).second)
            matching_pair_components_in(g, s, m, covering_pair(m, v))
            s.contains(covering_pair(m, v).first) and s.contains(covering_pair(m, v).second)
            g.adj(v, covering_pair(m, v).second)
            s.contains(v)
            neighborhood_of_set_contains_of_adj(g, s, t, v, covering_pair(m, v).second)
            neighborhood_of_set(g, s, t).contains(covering_pair(m, v).second)
            neighborhood_of_set(g, s, t).contains(matching_partner(m, v))
        }
        if not (covering_pair(m, v).first = v) {
            matching_partner(m, v) = covering_pair(m, v).first
            covering_pair(m, v).second = v
            is_matching_apply_edge(g, s, m, covering_pair(m, v))
            directed_edges(g, s).contains(covering_pair(m, v))
            directed_edges_adj(g, s, covering_pair(m, v))
            g.adj(covering_pair(m, v).first, covering_pair(m, v).second)
            g.adj(covering_pair(m, v).first, v)
            simple_graph_adj_symmetric(g, covering_pair(m, v).first, v)
            g.adj(v, covering_pair(m, v).first)
            matching_pair_components_in(g, s, m, covering_pair(m, v))
            s.contains(covering_pair(m, v).first) and s.contains(covering_pair(m, v).second)
            s.contains(v)
            neighborhood_of_set_contains_of_adj(g, s, t, v, covering_pair(m, v).first)
            neighborhood_of_set(g, s, t).contains(covering_pair(m, v).first)
            neighborhood_of_set(g, s, t).contains(matching_partner(m, v))
        }
        neighborhood_of_set(g, s, t).contains(matching_partner(m, v))
    }
}

/// The partner map is injective on a covered vertex set.
///
/// Two vertices with the same partner lie on the same matching pair (otherwise the
/// two pairs would share that partner vertex), and a pair determines its partner.
theorem matching_partner_locally_injective[V](g: SimpleGraph[V], s: FiniteSet[V], m: FiniteSet[Pair[V, V]], t: FiniteSet[V], v: V, w: V) {
    is_matching(g, s, m) and covers(m, t) and t.contains(v) and t.contains(w) and
        matching_partner(m, v) = matching_partner(m, w) implies v = w
} by {
    if is_matching(g, s, m) and covers(m, t) and t.contains(v) and t.contains(w) and
        matching_partner(m, v) = matching_partner(m, w) {
        matching_partner_pair_contains(m, t, v)
        m.contains(covering_pair(m, v))
        matching_partner_pair_contains(m, t, w)
        m.contains(covering_pair(m, w))
        matching_partner_pair_covers(m, t, v)
        covers_vertex_pred(m, v)(covering_pair(m, v)) =
            (m.contains(covering_pair(m, v)) and (covering_pair(m, v).first = v or covering_pair(m, v).second = v))
        covering_pair(m, v).first = v or covering_pair(m, v).second = v
        matching_partner_pair_covers(m, t, w)
        covers_vertex_pred(m, w)(covering_pair(m, w)) =
            (m.contains(covering_pair(m, w)) and (covering_pair(m, w).first = w or covering_pair(m, w).second = w))
        covering_pair(m, w).first = w or covering_pair(m, w).second = w
        if covering_pair(m, v).first = v {
            if covering_pair(m, w).first = w {
                matching_partner(m, v) = covering_pair(m, v).second
                matching_partner(m, w) = covering_pair(m, w).second
                covering_pair(m, v).second = covering_pair(m, w).second
                pairs_share_vertex(covering_pair(m, v), covering_pair(m, w)) =
                    (covering_pair(m, v).first = covering_pair(m, w).first or
                        covering_pair(m, v).first = covering_pair(m, w).second or
                        covering_pair(m, v).second = covering_pair(m, w).first or
                        covering_pair(m, v).second = covering_pair(m, w).second)
                pairs_share_vertex(covering_pair(m, v), covering_pair(m, w))
                is_matching_apply_disjoint(g, s, m, covering_pair(m, v), covering_pair(m, w))
                covering_pair(m, v) = covering_pair(m, w)
                covering_pair(m, v).first = covering_pair(m, w).first
                v = w
            }
            if not (covering_pair(m, w).first = w) {
                matching_partner(m, v) = covering_pair(m, v).second
                matching_partner(m, w) = covering_pair(m, w).first
                covering_pair(m, w).second = w
                covering_pair(m, v).second = covering_pair(m, w).first
                pairs_share_vertex(covering_pair(m, v), covering_pair(m, w)) =
                    (covering_pair(m, v).first = covering_pair(m, w).first or
                        covering_pair(m, v).first = covering_pair(m, w).second or
                        covering_pair(m, v).second = covering_pair(m, w).first or
                        covering_pair(m, v).second = covering_pair(m, w).second)
                pairs_share_vertex(covering_pair(m, v), covering_pair(m, w))
                is_matching_apply_disjoint(g, s, m, covering_pair(m, v), covering_pair(m, w))
                covering_pair(m, v) = covering_pair(m, w)
                covering_pair(m, v).first = covering_pair(m, w).first
                covering_pair(m, v).second = covering_pair(m, w).second
                covering_pair(m, v).second = covering_pair(m, v).first
                matching_pair_ne(g, s, m, covering_pair(m, v))
                covering_pair(m, v).first != covering_pair(m, v).second
                false
            }
            v = w
        }
        if not (covering_pair(m, v).first = v) {
            covering_pair(m, v).second = v
            if covering_pair(m, w).first = w {
                matching_partner(m, v) = covering_pair(m, v).first
                matching_partner(m, w) = covering_pair(m, w).second
                covering_pair(m, v).first = covering_pair(m, w).second
                pairs_share_vertex(covering_pair(m, v), covering_pair(m, w)) =
                    (covering_pair(m, v).first = covering_pair(m, w).first or
                        covering_pair(m, v).first = covering_pair(m, w).second or
                        covering_pair(m, v).second = covering_pair(m, w).first or
                        covering_pair(m, v).second = covering_pair(m, w).second)
                pairs_share_vertex(covering_pair(m, v), covering_pair(m, w))
                is_matching_apply_disjoint(g, s, m, covering_pair(m, v), covering_pair(m, w))
                covering_pair(m, v) = covering_pair(m, w)
                covering_pair(m, v).first = covering_pair(m, w).first
                covering_pair(m, v).second = covering_pair(m, w).second
                covering_pair(m, v).first = covering_pair(m, v).second
                matching_pair_ne(g, s, m, covering_pair(m, v))
                covering_pair(m, v).first != covering_pair(m, v).second
                false
            }
            if not (covering_pair(m, w).first = w) {
                matching_partner(m, v) = covering_pair(m, v).first
                matching_partner(m, w) = covering_pair(m, w).first
                covering_pair(m, w).second = w
                covering_pair(m, v).first = covering_pair(m, w).first
                pairs_share_vertex(covering_pair(m, v), covering_pair(m, w)) =
                    (covering_pair(m, v).first = covering_pair(m, w).first or
                        covering_pair(m, v).first = covering_pair(m, w).second or
                        covering_pair(m, v).second = covering_pair(m, w).first or
                        covering_pair(m, v).second = covering_pair(m, w).second)
                pairs_share_vertex(covering_pair(m, v), covering_pair(m, w))
                is_matching_apply_disjoint(g, s, m, covering_pair(m, v), covering_pair(m, w))
                covering_pair(m, v) = covering_pair(m, w)
                covering_pair(m, v).second = covering_pair(m, w).second
                v = w
            }
            v = w
        }
        v = w
    }
}

/// The partner map of a matching, as a function on vertices.
define matching_partner_fn[V](m: FiniteSet[Pair[V, V]]) -> (V -> V) {
    function(v: V) {
        matching_partner(m, v)
    }
}

/// A matching that covers a vertex set forces the Hall condition on it.
///
/// Each vertex of the covered set is matched to a distinct neighbor, so the covered
/// set injects into its neighborhood. This is the necessary direction of Hall's
/// marriage theorem; it needs no bipartiteness.
theorem matching_covering_neighborhood_bound[V](g: SimpleGraph[V], s: FiniteSet[V], m: FiniteSet[Pair[V, V]], t: FiniteSet[V]) {
    is_matching(g, s, m) and covers(m, t) implies
        fs_card(t) <= fs_card(neighborhood_of_set(g, s, t))
} by {
    if is_matching(g, s, m) and covers(m, t) {
        forall(x: V, y: V) {
            if t.contains(x) and t.contains(y) and matching_partner_fn(m)(x) = matching_partner_fn(m)(y) {
                matching_partner_fn(m)(x) = matching_partner(m, x)
                matching_partner_fn(m)(y) = matching_partner(m, y)
                matching_partner(m, x) = matching_partner(m, y)
                matching_partner_locally_injective(g, s, m, t, x, y)
                x = y
            }
            t.contains(x) and t.contains(y) and matching_partner_fn(m)(x) = matching_partner_fn(m)(y) implies x = y
        }
        fs_card_image_eq_of_locally_injective(t, matching_partner_fn(m))
        fs_card(fs_image(t, matching_partner_fn(m))) = fs_card(t)
        forall(v: V) {
            if fs_image(t, matching_partner_fn(m)).contains(v) {
                finite_set_image_contains_witness(t, matching_partner_fn(m), v)
                let x: V satisfy {
                    t.contains(x) and v = matching_partner_fn(m)(x)
                }
                matching_partner_fn(m)(x) = matching_partner(m, x)
                v = matching_partner(m, x)
                matching_partner_in_neighborhood(g, s, m, t, x)
                neighborhood_of_set(g, s, t).contains(matching_partner(m, x))
                neighborhood_of_set(g, s, t).contains(v)
            }
            fs_image(t, matching_partner_fn(m)).contains(v) implies neighborhood_of_set(g, s, t).contains(v)
        }
        fs_subset_eq_intro(fs_image(t, matching_partner_fn(m)), neighborhood_of_set(g, s, t))
        fs_image(t, matching_partner_fn(m)).subset_eq(neighborhood_of_set(g, s, t))
        fs_card_mono(fs_image(t, matching_partner_fn(m)), neighborhood_of_set(g, s, t))
        fs_card(fs_image(t, matching_partner_fn(m))) <= fs_card(neighborhood_of_set(g, s, t))
        fs_card(t) <= fs_card(neighborhood_of_set(g, s, t))
    }
}

/// The necessary direction of Hall's marriage theorem.
///
/// In a graph with bipartition `part` on the vertex set `s`, a matching covering the
/// side `bipartite_side(s, part)` forces every subset of that side to have at least
/// as many neighbors as elements.
theorem hall_necessary_condition[V](g: SimpleGraph[V], s: FiniteSet[V], part: V -> Bool) {
    is_bipartition(g, s, part) and exists(m: FiniteSet[Pair[V, V]]) {
        is_matching(g, s, m) and covers(m, bipartite_side(s, part))
    } implies forall(t: FiniteSet[V]) {
        t.subset_eq(bipartite_side(s, part)) implies
            fs_card(t) <= fs_card(neighborhood_of_set(g, s, t))
    }
} by {
    if is_bipartition(g, s, part) and exists(m: FiniteSet[Pair[V, V]]) {
        is_matching(g, s, m) and covers(m, bipartite_side(s, part))
    } {
        let m: FiniteSet[Pair[V, V]] satisfy {
            is_matching(g, s, m) and covers(m, bipartite_side(s, part))
        }
        forall(t: FiniteSet[V]) {
            if t.subset_eq(bipartite_side(s, part)) {
                covers_of_subset(m, t, bipartite_side(s, part))
                covers(m, t)
                matching_covering_neighborhood_bound(g, s, m, t)
                fs_card(t) <= fs_card(neighborhood_of_set(g, s, t))
            }
            (t.subset_eq(bipartite_side(s, part)) implies
                fs_card(t) <= fs_card(neighborhood_of_set(g, s, t)))
        }
    }
}

// Hall's marriage theorem for bipartite graphs.
//
// Let `g` be a simple graph on the finite vertex set `s` with bipartition `part`,
// and let X = bipartite_side(s, part) be one side. A matching covering X exists if
// and only if every subset `t` of X has at least as many neighbors as elements:
//
//   exists(m) { is_matching(g, s, m) and covers(m, X) }
//     = forall(t) { t.subset_eq(X) implies fs_card(t) <= fs_card(neighborhood_of_set(g, s, t)) }
//
// The necessary direction is proved above as `hall_necessary_condition` (indeed
// `matching_covering_neighborhood_bound` holds for every graph, bipartite or not).
// The sufficiency needs an induction on the size of X, splitting on whether a proper
// subset is tight; it is left for future work. The full statement is recorded here
// with the library's bipartite API.
//
// theorem hall_marriage_theorem[V](g: SimpleGraph[V], s: FiniteSet[V], part: V -> Bool) {
//     is_bipartition(g, s, part) implies
//     (exists(m: FiniteSet[Pair[V, V]]) {
//         is_matching(g, s, m) and covers(m, bipartite_side(s, part))
//     }) = (forall(t: FiniteSet[V]) {
//         t.subset_eq(bipartite_side(s, part)) implies
//             fs_card(t) <= fs_card(neighborhood_of_set(g, s, t))
//     })
// }
