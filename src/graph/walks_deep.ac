/// Deeper graph-walk results: concatenation and reversal of walks, the distance function
/// (the shortest walk length, with the adjacent case), and the walk-contains-a-path
/// theorem. The odd closed walk to odd cycle extraction is stated but open.

from nat import Nat, pos_of_ne_zero, lt_imp_lte_suc
from list import List
from graph.simple_graph import SimpleGraph, simple_graph_adj_ne
from graph.simple_graph_walks import simple_graph_walk, simple_graph_walk_nil,
    simple_graph_walk_of_adj, simple_graph_path
from graph.simple_graph_bipartite_odd_cycle import simple_graph_walk_concat,
    simple_graph_walk_reverse, rev_walk_targets
from graph.graph_structure import walk_has_path
from graph.simple_graph_forest import length_zero_imp_nil

numerals Nat

// ============================================================================
// (a) Walk concatenation.
// ============================================================================

/// A walk from `a` to `m` followed by a walk from `m` to `b` is a walk from `a` to `b`.
///
/// This is the composition of the explicit walk API: the target list of the combined
/// walk is the concatenation of the two target lists. The statement is the one proved
/// in `simple_graph_bipartite_odd_cycle.ac` (`simple_graph_walk_concat`), restated here
/// as part of the walks-deepening layer.
theorem walk_concat[V](g: SimpleGraph[V], a: V, m: V, b: V, w1: List[V], w2: List[V]) {
    simple_graph_walk(g, a, m, w1) and simple_graph_walk(g, m, b, w2)
        implies simple_graph_walk(g, a, b, w1 + w2)
} by {
    simple_graph_walk_concat(g, a, m, b, w1, w2)
    simple_graph_walk(g, a, m, w1) and simple_graph_walk(g, m, b, w2) implies simple_graph_walk(g, a, b, w1 + w2)
}

// ============================================================================
// (b) Walk reversal.
// ============================================================================

/// The reverse of a walk is a walk back to its start.
///
/// Reversing a walk `a -> ... -> b` with target list `steps` gives a walk `b -> ... -> a`
/// whose target list is `rev_walk_targets(steps, a)`. This restates the theorem proved
/// in `simple_graph_bipartite_odd_cycle.ac` (`simple_graph_walk_reverse`).
theorem walk_reverse[V](g: SimpleGraph[V], a: V, b: V, steps: List[V]) {
    simple_graph_walk(g, a, b, steps)
        implies simple_graph_walk(g, b, a, rev_walk_targets(steps, a))
} by {
    simple_graph_walk_reverse(g, a, b, steps)
    simple_graph_walk(g, a, b, steps) implies simple_graph_walk(g, b, a, rev_walk_targets(steps, a))
}

// ============================================================================
// (d) The distance function.
// ============================================================================

/// True when `n` is the length of a shortest walk from `x` to `y`: a walk of length `n`
/// exists, and every walk between the two vertices has length at least `n`.
define graph_distance[V](g: SimpleGraph[V], x: V, y: V, n: Nat) -> Bool {
    exists(steps: List[V]) {
        simple_graph_walk(g, x, y, steps) and steps.length = n and
            forall(shorter: List[V]) {
                simple_graph_walk(g, x, y, shorter) implies n <= shorter.length
            }
    }
}

// The distance is the shortest walk length between connected vertices: reachable vertices
// have at least one walk, and the naturals are well-ordered, so a minimal length exists.
// The well-ordering step (a minimum of a nonempty set of walk lengths) needs bounded
// induction machinery that is not yet a standalone lemma, so the statement is left open:
//   theorem graph_distance_of_reachable[V](g: SimpleGraph[V], x: V, y: V) {
//       simple_graph_reachable(g, x, y) implies exists(n: Nat) {
//           graph_distance(g, x, y, n)
//       }
//   }

/// Adjacent vertices are at distance one: the one-step walk has length one, and no
/// shorter walk exists because a length-zero walk would identify the two endpoints.
theorem graph_distance_adj[V](g: SimpleGraph[V], x: V, y: V) {
    g.adj(x, y) implies graph_distance(g, x, y, Nat.1)
} by {
    if g.adj(x, y) {
        forall(shorter: List[V]) {
            if simple_graph_walk(g, x, y, shorter) {
                if shorter.length = Nat.0 {
                    length_zero_imp_nil(shorter)
                    shorter = List.nil[V]
                    simple_graph_walk_nil(g, x, y)
                    x = y
                    simple_graph_adj_ne(g, x, y)
                    g.adj(x, y) implies x != y
                    x != y
                    false
                }
                shorter.length != Nat.0
                pos_of_ne_zero(shorter.length)
                Nat.0 < shorter.length
                lt_imp_lte_suc(Nat.0, shorter.length)
                Nat.0.suc <= shorter.length
                Nat.1 <= shorter.length
            }
            simple_graph_walk(g, x, y, shorter) implies Nat.1 <= shorter.length
        }
        simple_graph_walk_of_adj(g, x, y)
        simple_graph_walk(g, x, y, List.singleton(y))
        List.singleton(y) = List.cons(y, List.nil[V])
        List.cons(y, List.nil[V]).length = List.nil[V].length.suc
        List.nil[V].length = Nat.0
        List.singleton(y).length = Nat.1
        graph_distance(g, x, y, Nat.1) = exists(steps: List[V]) {
            simple_graph_walk(g, x, y, steps) and steps.length = Nat.1 and
                forall(shorter2: List[V]) {
                    simple_graph_walk(g, x, y, shorter2) implies Nat.1 <= shorter2.length
                }
        }
        exists(steps: List[V]) {
            steps = List.singleton(y) and simple_graph_walk(g, x, y, steps) and
                steps.length = Nat.1 and
                forall(shorter2: List[V]) {
                    simple_graph_walk(g, x, y, shorter2) implies Nat.1 <= shorter2.length
                }
        }
        graph_distance(g, x, y, Nat.1)
    }
}

// ============================================================================
// (e) Every walk contains a path.
// ============================================================================

/// Every walk contains a path between its endpoints.
///
/// A walk with a repeated vertex is shortened by cutting out the loop between two
/// occurrences of the vertex, so a path (a walk with no repeated vertices) remains.
/// This restates `walk_has_path` from `graph_structure.ac`.
theorem walk_contains_path[V](g: SimpleGraph[V], a: V, b: V, steps: List[V]) {
    simple_graph_walk(g, a, b, steps) implies exists(steps2: List[V]) {
        simple_graph_path(g, a, b, steps2)
    }
} by {
    walk_has_path(g, a, b, steps)
    simple_graph_walk(g, a, b, steps) implies exists(steps2: List[V]) {
        simple_graph_path(g, a, b, steps2)
    }
}

// ============================================================================
// (c) An odd closed walk contains an odd cycle (statement).
// ============================================================================

// A closed walk of odd length contains an odd cycle: cut the closed walk at the first
// repeated vertex and keep the shorter odd piece between the two occurrences. The
// extraction needs list-splitting machinery (locate a repeated target, split the walk
// there, and check the parity and uniqueness of the piece), which is left open in
// `simple_graph_bipartite_odd_cycle.ac` — see the commented `is_bipartite_of_no_odd_cycle`
// there, whose missing ingredient is exactly this lemma. With it, `has_odd_cycle`
// follows from `has_odd_closed_walk`:
//
//   theorem odd_closed_walk_contains_odd_cycle[V](g: SimpleGraph[V], s: FiniteSet[V]) {
//       has_odd_closed_walk(g, s) implies has_odd_cycle(g, s)
//   }
