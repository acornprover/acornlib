from nat import Nat
from finite_set import FiniteSet, finite_set_subset_contains
from graph.simple_graph import SimpleGraph, complete_graph, complete_graph_adj_iff_ne
from graph.simple_graph_vertex_sets import common_neighborhood, common_neighborhood_contains_eq
from graph.simple_graph_triangle_free import is_triangle_free, is_triangle_free_apply,
    has_no_common_neighbor, has_no_common_neighbor_apply

numerals Nat

/// True if `x` and `y` have two distinct non-adjacent common neighbors in `s`.
///
/// Named so that diamond-freeness below is a two-variable statement. Stated directly, a
/// diamond is a four-variable condition, and neither its restriction to subsets nor its
/// relation to triangle-freeness goes through at that depth.
define has_nonadjacent_common_neighbors[V](
    g: SimpleGraph[V], s: FiniteSet[V], x: V, y: V
) -> Bool {
    exists(z: V, w: V) {
        common_neighborhood(g, s, x, y).contains(z)
            and common_neighborhood(g, s, x, y).contains(w)
            and z != w and not g.adj(z, w)
    }
}

/// Two such common neighbors witness the condition.
theorem has_nonadjacent_common_neighbors_intro[V](
    g: SimpleGraph[V], s: FiniteSet[V], x: V, y: V, z: V, w: V
) {
    common_neighborhood(g, s, x, y).contains(z)
        and common_neighborhood(g, s, x, y).contains(w)
        and z != w and not g.adj(z, w)
        implies has_nonadjacent_common_neighbors(g, s, x, y)
} by {
    if common_neighborhood(g, s, x, y).contains(z)
        and common_neighborhood(g, s, x, y).contains(w)
        and z != w and not g.adj(z, w) {
        has_nonadjacent_common_neighbors(g, s, x, y) = exists(a: V, b: V) {
            common_neighborhood(g, s, x, y).contains(a)
                and common_neighborhood(g, s, x, y).contains(b)
                and a != b and not g.adj(a, b)
        }
        exists(a: V, b: V) {
            common_neighborhood(g, s, x, y).contains(a)
                and common_neighborhood(g, s, x, y).contains(b)
                and a != b and not g.adj(a, b)
        }
        has_nonadjacent_common_neighbors(g, s, x, y)
    }
}

/// Two such common neighbors can be extracted.
theorem has_nonadjacent_common_neighbors_witness[V](
    g: SimpleGraph[V], s: FiniteSet[V], x: V, y: V
) {
    has_nonadjacent_common_neighbors(g, s, x, y) implies exists(z: V, w: V) {
        common_neighborhood(g, s, x, y).contains(z)
            and common_neighborhood(g, s, x, y).contains(w)
            and z != w and not g.adj(z, w)
    }
} by {
    if has_nonadjacent_common_neighbors(g, s, x, y) {
        has_nonadjacent_common_neighbors(g, s, x, y) = exists(a: V, b: V) {
            common_neighborhood(g, s, x, y).contains(a)
                and common_neighborhood(g, s, x, y).contains(b)
                and a != b and not g.adj(a, b)
        }
        exists(a: V, b: V) {
            common_neighborhood(g, s, x, y).contains(a)
                and common_neighborhood(g, s, x, y).contains(b)
                and a != b and not g.adj(a, b)
        }
    }
}

/// True if no four vertices of `s` induce a diamond.
///
/// A diamond is a four-cycle with one chord: an adjacent pair together with two of their
/// common neighbors that are not adjacent to each other. Diamond-free graphs are exactly
/// those in which every edge lies in at most one maximal clique.
define is_diamond_free[V](g: SimpleGraph[V], s: FiniteSet[V]) -> Bool {
    forall(x: V, y: V) {
        (s.contains(x) and s.contains(y) and g.adj(x, y)
            implies not has_nonadjacent_common_neighbors(g, s, x, y))
    }
}

/// An adjacent pair in a diamond-free graph has no two non-adjacent common neighbors.
theorem is_diamond_free_apply[V](
    g: SimpleGraph[V], s: FiniteSet[V], x: V, y: V
) {
    is_diamond_free(g, s) and s.contains(x) and s.contains(y) and g.adj(x, y)
        implies not has_nonadjacent_common_neighbors(g, s, x, y)
} by {
    if is_diamond_free(g, s) and s.contains(x) and s.contains(y) and g.adj(x, y) {
        is_diamond_free(g, s) = forall(u: V, v: V) {
            (s.contains(u) and s.contains(v) and g.adj(u, v)
                implies not has_nonadjacent_common_neighbors(g, s, u, v))
        }
        forall(u: V, v: V) {
            (s.contains(u) and s.contains(v) and g.adj(u, v)
                implies not has_nonadjacent_common_neighbors(g, s, u, v))
        }
        (s.contains(x) and s.contains(y) and g.adj(x, y)
            implies not has_nonadjacent_common_neighbors(g, s, x, y))
        not has_nonadjacent_common_neighbors(g, s, x, y)
    }
}

/// The pointwise condition is diamond-freeness.
theorem is_diamond_free_intro[V](g: SimpleGraph[V], s: FiniteSet[V]) {
    (forall(x: V, y: V) {
        (s.contains(x) and s.contains(y) and g.adj(x, y)
            implies not has_nonadjacent_common_neighbors(g, s, x, y))
    }) implies is_diamond_free(g, s)
} by {
    if forall(x: V, y: V) {
        (s.contains(x) and s.contains(y) and g.adj(x, y)
            implies not has_nonadjacent_common_neighbors(g, s, x, y))
    } {
        is_diamond_free(g, s) = forall(u: V, v: V) {
            (s.contains(u) and s.contains(v) and g.adj(u, v)
                implies not has_nonadjacent_common_neighbors(g, s, u, v))
        }
        is_diamond_free(g, s)
    }
}

/// A diamond-free graph has no four vertices in the classical diamond configuration.
///
/// This is the reading in terms of the four vertices themselves, recovered from the
/// common-neighbor form.
theorem is_diamond_free_no_diamond[V](
    g: SimpleGraph[V], s: FiniteSet[V], x: V, y: V, z: V, w: V
) {
    is_diamond_free(g, s) and s.contains(x) and s.contains(y) and s.contains(z)
        and s.contains(w) and g.adj(x, y) and g.adj(x, z) and g.adj(y, z)
        and g.adj(x, w) and g.adj(y, w) and z != w
        implies g.adj(z, w)
} by {
    if is_diamond_free(g, s) and s.contains(x) and s.contains(y) and s.contains(z)
        and s.contains(w) and g.adj(x, y) and g.adj(x, z) and g.adj(y, z)
        and g.adj(x, w) and g.adj(y, w) and z != w {
        if not g.adj(z, w) {
            common_neighborhood_contains_eq(g, s, x, y, z)
            common_neighborhood(g, s, x, y).contains(z)
            common_neighborhood_contains_eq(g, s, x, y, w)
            common_neighborhood(g, s, x, y).contains(w)
            has_nonadjacent_common_neighbors_intro(g, s, x, y, z, w)
            has_nonadjacent_common_neighbors(g, s, x, y)
            is_diamond_free_apply(g, s, x, y)
            not has_nonadjacent_common_neighbors(g, s, x, y)
            false
        }
        g.adj(z, w)
    }
}

/// Diamond-freeness passes to subsets of the vertex set.
theorem is_diamond_free_of_subset[V](
    g: SimpleGraph[V], s: FiniteSet[V], r: FiniteSet[V]
) {
    r.subset_eq(s) and is_diamond_free(g, s) implies is_diamond_free(g, r)
} by {
    if r.subset_eq(s) and is_diamond_free(g, s) {
        forall(x: V, y: V) {
            if r.contains(x) and r.contains(y) and g.adj(x, y) {
                if has_nonadjacent_common_neighbors(g, r, x, y) {
                    has_nonadjacent_common_neighbors_witness(g, r, x, y)
                    let (z: V, w: V) satisfy {
                        common_neighborhood(g, r, x, y).contains(z)
                            and common_neighborhood(g, r, x, y).contains(w)
                            and z != w and not g.adj(z, w)
                    }
                    common_neighborhood_contains_eq(g, r, x, y, z)
                    r.contains(z)
                    finite_set_subset_contains(r, s, z)
                    s.contains(z)
                    common_neighborhood_contains_eq(g, r, x, y, w)
                    r.contains(w)
                    finite_set_subset_contains(r, s, w)
                    s.contains(w)
                    finite_set_subset_contains(r, s, x)
                    s.contains(x)
                    finite_set_subset_contains(r, s, y)
                    s.contains(y)
                    common_neighborhood_contains_eq(g, s, x, y, z)
                    common_neighborhood(g, s, x, y).contains(z)
                    common_neighborhood_contains_eq(g, s, x, y, w)
                    common_neighborhood(g, s, x, y).contains(w)
                    has_nonadjacent_common_neighbors_intro(g, s, x, y, z, w)
                    has_nonadjacent_common_neighbors(g, s, x, y)
                    is_diamond_free_apply(g, s, x, y)
                    not has_nonadjacent_common_neighbors(g, s, x, y)
                    false
                }
                not has_nonadjacent_common_neighbors(g, r, x, y)
            }
            (r.contains(x) and r.contains(y) and g.adj(x, y)
                implies not has_nonadjacent_common_neighbors(g, r, x, y))
        }
        is_diamond_free_intro(g, r)
        is_diamond_free(g, r)
    }
}

/// A triangle-free graph is diamond-free.
///
/// A diamond contains a triangle, so there is nothing left to forbid: an adjacent pair in a
/// triangle-free graph has no common neighbor at all, let alone two.
theorem triangle_free_is_diamond_free[V](g: SimpleGraph[V], s: FiniteSet[V]) {
    is_triangle_free(g, s) implies is_diamond_free(g, s)
} by {
    if is_triangle_free(g, s) {
        forall(x: V, y: V) {
            if s.contains(x) and s.contains(y) and g.adj(x, y) {
                if has_nonadjacent_common_neighbors(g, s, x, y) {
                    has_nonadjacent_common_neighbors_witness(g, s, x, y)
                    let (z: V, w: V) satisfy {
                        common_neighborhood(g, s, x, y).contains(z)
                            and common_neighborhood(g, s, x, y).contains(w)
                            and z != w and not g.adj(z, w)
                    }
                    is_triangle_free_apply(g, s, x, y)
                    has_no_common_neighbor(g, s, x, y)
                    has_no_common_neighbor_apply(g, s, x, y, z)
                    not common_neighborhood(g, s, x, y).contains(z)
                    false
                }
                not has_nonadjacent_common_neighbors(g, s, x, y)
            }
            (s.contains(x) and s.contains(y) and g.adj(x, y)
                implies not has_nonadjacent_common_neighbors(g, s, x, y))
        }
        is_diamond_free_intro(g, s)
        is_diamond_free(g, s)
    }
}

/// The complete graph is diamond-free.
///
/// Two distinct vertices are always adjacent there, so no two common neighbors can fail to
/// be adjacent.
theorem complete_graph_is_diamond_free[V](s: FiniteSet[V]) {
    is_diamond_free(complete_graph[V], s)
} by {
    forall(x: V, y: V) {
        if s.contains(x) and s.contains(y) and complete_graph[V].adj(x, y) {
            if has_nonadjacent_common_neighbors(complete_graph[V], s, x, y) {
                has_nonadjacent_common_neighbors_witness(complete_graph[V], s, x, y)
                let (z: V, w: V) satisfy {
                    common_neighborhood(complete_graph[V], s, x, y).contains(z)
                        and common_neighborhood(complete_graph[V], s, x, y).contains(w)
                        and z != w and not complete_graph[V].adj(z, w)
                }
                complete_graph_adj_iff_ne(z, w)
                complete_graph[V].adj(z, w) = (z != w)
                complete_graph[V].adj(z, w)
                false
            }
            not has_nonadjacent_common_neighbors(complete_graph[V], s, x, y)
        }
        (s.contains(x) and s.contains(y) and complete_graph[V].adj(x, y)
            implies not has_nonadjacent_common_neighbors(complete_graph[V], s, x, y))
    }
    is_diamond_free_intro(complete_graph[V], s)
    is_diamond_free(complete_graph[V], s)
}
