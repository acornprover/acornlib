/// Public interface for graph theory.

from data.basic.functions import binary_function_extensionality, compose,
    compose_injective_fn, identity_fn, injective_fn_ne, is_injective_fn,
    is_left_inverse_fn, left_inverse_fn_imp_injective_fn
from data.basic.logic import exists_intro, false_implies
from data.basic.relation_basic import is_irreflexive, is_symmetric
from data.basic.set import Set, image_contains, maps_into_set_image, set_image,
    set_image_contains_eq, set_preimage, set_preimage_contains_eq, subset_contains
from data.finite.finite_set_card import fs_card, fs_card_empty
from data.finite.finite_set_card_ops import fs_card_insert_of_not_contains,
    fs_card_remove_of_contains
from data.finite.finite_set_membership import fs_insert_contains_cases,
    fs_insert_contains_eq, fs_remove_contains_eq
from data.finite.finite_set_unique_induction import finite_set_strong_induction
from data.list.list_cons_membership import cons_contains_head,
    cons_contains_of_tail_contains, nil_not_contains
from data.list.list_reverse_append_helpers import reverse_cons_eq_append, reverse_nil
from data.nat.nat_range_set import range_set, range_set_contains, range_set_contains_eq
from finite_set import FiniteSet, finite_set_empty_contains_eq, finite_set_ext_contains,
    finite_set_sum, finite_set_sum_empty, finite_set_sum_insert, fs_insert, fs_remove
from list import List, add_length, reverse, reverse_length
from nat import Nat, add_comm, add_imp_sub, distrib_left, div_mul, lt_and_lte, mul_comm,
    mul_one_right, mul_zero_right, pos_of_ne_zero, suc_sub_one
from pair import Pair

structure SimpleGraph[V] {
    /// True if the two vertices are adjacent.
    adj: (V, V) -> Bool
} constraint {
    is_symmetric(adj) and is_irreflexive(adj)
}

define is_graph_hom[V, W](g: SimpleGraph[V], h: SimpleGraph[W], f: V -> W) -> Bool {
    forall(x: V, y: V) {
        g.adj(x, y) implies h.adj(f(x), f(y))
    }
}

define reflects_graph_adj[V, W](g: SimpleGraph[V], h: SimpleGraph[W], f: V -> W) -> Bool {
    forall(x: V, y: V) {
        h.adj(f(x), f(y)) implies g.adj(x, y)
    }
}

define is_graph_embedding[V, W](g: SimpleGraph[V], h: SimpleGraph[W], f: V -> W) -> Bool {
    is_graph_hom(g, h, f) and reflects_graph_adj(g, h, f) and is_injective_fn(f)
}

structure SimpleGraphEmbedding[V, W] {
    /// The source graph.
    src: SimpleGraph[V]

    /// The target graph.
    dst: SimpleGraph[W]

    /// The map on vertices.
    map: V -> W
} constraint {
    is_graph_embedding(src, dst, map)
}

structure SimpleGraphHom[V, W] {
    /// The source graph.
    src: SimpleGraph[V]

    /// The target graph.
    dst: SimpleGraph[W]

    /// The map on vertices.
    map: V -> W
} constraint {
    is_graph_hom(src, dst, map)
}

define are_inverse_vertex_maps[V, W](f: V -> W, e: W -> V) -> Bool {
    (forall(x: V) { e(f(x)) = x }) and (forall(y: W) { f(e(y)) = y })
}

define is_graph_iso_pair[V, W](
    g: SimpleGraph[V],
    h: SimpleGraph[W],
    f: V -> W,
    e: W -> V
) -> Bool {
    is_graph_hom(g, h, f) and is_graph_hom(h, g, e) and are_inverse_vertex_maps(f, e)
}

structure SimpleGraphIso[V, W] {
    /// The source graph.
    src: SimpleGraph[V]

    /// The target graph.
    dst: SimpleGraph[W]

    /// The forward map on vertices.
    map: V -> W

    /// The inverse map on vertices.
    inv: W -> V
} constraint {
    is_graph_iso_pair(src, dst, map, inv)
}


numerals Nat
theorem bool_ne_false(p: Bool) {
    (p != false) = p
}

theorem bool_ne_not(p: Bool, q: Bool) {
    not (p != q) = (p != not q)
}

define complete_graph_adj[V](x: V, y: V) -> Bool {
    x != y
}

let complete_graph[V]: SimpleGraph[V] satisfy {
    SimpleGraph.new(complete_graph_adj[V]) = Option.some(complete_graph)
}

define graph_complement_adj[V](g: SimpleGraph[V], x: V, y: V) -> Bool {
    x != y and not g.adj(x, y)
}

let graph_complement[V](g: SimpleGraph[V]) -> result: SimpleGraph[V] satisfy {
    SimpleGraph.new(graph_complement_adj(g)) = Option.some(result)
}

define graph_intersection_adj[V](g: SimpleGraph[V], h: SimpleGraph[V], x: V, y: V) -> Bool {
    g.adj(x, y) and h.adj(x, y)
}

let graph_intersection[V](g: SimpleGraph[V], h: SimpleGraph[V]) -> result: SimpleGraph[V] satisfy {
    SimpleGraph.new(graph_intersection_adj(g, h)) = Option.some(result)
}

define graph_union_adj[V](g: SimpleGraph[V], h: SimpleGraph[V], x: V, y: V) -> Bool {
    g.adj(x, y) or h.adj(x, y)
}

numerals Nat
define has_blue_triangle[V](color: (V, V) -> Bool, s: FiniteSet[V]) -> Bool {
    exists(x: V, y: V, z: V) {
        s.contains(x) and s.contains(y) and s.contains(z) and x != y and x != z and y != z
        and not color(x, y) and not color(x, z) and not color(y, z)
    }
}

theorem has_blue_triangle_intro[V](color: (V, V) -> Bool, s: FiniteSet[V], x: V, y: V, z: V) {
    s.contains(x) and s.contains(y) and s.contains(z) and x != y and x != z and y != z
    and not color(x, y) and not color(x, z) and not color(y, z)
    implies has_blue_triangle(color, s)
}

define has_red_triangle[V](color: (V, V) -> Bool, s: FiniteSet[V]) -> Bool {
    exists(x: V, y: V, z: V) {
        s.contains(x) and s.contains(y) and s.contains(z) and x != y and x != z and y != z
        and color(x, y) and color(x, z) and color(y, z)
    }
}

define has_mono_triangle[V](color: (V, V) -> Bool, s: FiniteSet[V]) -> Bool {
    has_red_triangle(color, s) or has_blue_triangle(color, s)
}

theorem has_red_triangle_intro[V](color: (V, V) -> Bool, s: FiniteSet[V], x: V, y: V, z: V) {
    s.contains(x) and s.contains(y) and s.contains(z) and x != y and x != z and y != z
    and color(x, y) and color(x, z) and color(y, z)
    implies has_red_triangle(color, s)
}

define induced_subgraph_adj[V](g: SimpleGraph[V], s: Set[V], x: V, y: V) -> Bool {
    g.adj(x, y) and s.contains(x) and s.contains(y)
}

let induced_subgraph[V](g: SimpleGraph[V], s: Set[V]) -> result: SimpleGraph[V] satisfy {
    SimpleGraph.new(induced_subgraph_adj(g, s)) = Option.some(result)
}

define is_clique[V](g: SimpleGraph[V], s: Set[V]) -> Bool {
    forall(x: V, y: V) {
        s.contains(x) and s.contains(y) and x != y implies g.adj(x, y)
    }
}

numerals Nat
define is_edge_2_coloring[V](g: SimpleGraph[V], color: (V, V) -> Bool) -> Bool {
    forall(x: V, y: V) {
        g.adj(x, y) implies color(x, y) = color(y, x)
    }
}

theorem is_graph_iso_pair_is_graph_embedding[V, W](
    g: SimpleGraph[V], h: SimpleGraph[W], f: V -> W, e: W -> V
) {
    is_graph_iso_pair(g, h, f, e) implies is_graph_embedding(g, h, f)
}

theorem is_graph_iso_pair_map_is_injective[V, W](
    g: SimpleGraph[V], h: SimpleGraph[W], f: V -> W, e: W -> V
) {
    is_graph_iso_pair(g, h, f, e) implies is_injective_fn(f)
}

theorem is_graph_iso_pair_reflects_adj[V, W](
    g: SimpleGraph[V], h: SimpleGraph[W], f: V -> W, e: W -> V
) {
    is_graph_iso_pair(g, h, f, e) implies reflects_graph_adj(g, h, f)
}

define is_independent_set[V](g: SimpleGraph[V], s: Set[V]) -> Bool {
    forall(x: V, y: V) {
        s.contains(x) and s.contains(y) and x != y implies not g.adj(x, y)
    }
}

numerals Nat
define is_planar[V](g: SimpleGraph[V]) -> Bool {
    false
}

define nat_odd(n: Nat) -> Bool {
    match n {
        Nat.zero {
            false
        }
        Nat.suc(pred) {
            not nat_odd(pred)
        }
    }
}

theorem nat_odd_add(a: Nat, b: Nat) {
    nat_odd(a + b) = (nat_odd(a) != nat_odd(b))
}

let six_vertices: FiniteSet[Nat] = range_set(Nat.6)

/// The five vertices `{0, ..., 4}` on which the lower bound lives.

theorem ramsey_r33_upper(color: (Nat, Nat) -> Bool) {
    is_edge_2_coloring(complete_graph[Nat], color) implies
    has_mono_triangle(color, six_vertices)
}

define simple_graph_coloring[V, C](g: SimpleGraph[V], color: V -> C) -> Bool {
    forall(x: V, y: V) {
        g.adj(x, y) implies color(x) != color(y)
    }
}

define simple_graph_embedding_image_set[V, W](f: SimpleGraphEmbedding[V, W], s: Set[V]) -> Set[W] {
    set_image(s, f.map)
}

define simple_graph_embedding_image[V, W](f: SimpleGraphEmbedding[V, W], s: Set[V]) -> SimpleGraph[W] {
    induced_subgraph(f.dst, simple_graph_embedding_image_set(f, s))
}

theorem simple_graph_embedding_new_dst[V, W](
    g: SimpleGraph[V],
    h: SimpleGraph[W],
    f: V -> W,
    p: SimpleGraphEmbedding[V, W]
) {
    SimpleGraphEmbedding[V, W].new(g, h, f) = Option.some(p) implies p.dst = h
}

theorem simple_graph_embedding_new_map[V, W](
    g: SimpleGraph[V],
    h: SimpleGraph[W],
    f: V -> W,
    p: SimpleGraphEmbedding[V, W]
) {
    SimpleGraphEmbedding[V, W].new(g, h, f) = Option.some(p) implies p.map = f
}

theorem simple_graph_embedding_new_src[V, W](
    g: SimpleGraph[V],
    h: SimpleGraph[W],
    f: V -> W,
    p: SimpleGraphEmbedding[V, W]
) {
    SimpleGraphEmbedding[V, W].new(g, h, f) = Option.some(p) implies p.src = g
}

let simple_graph_embedding_of_iso[V, W](f: SimpleGraphIso[V, W]) -> result: SimpleGraphEmbedding[V, W] satisfy {
    result.src = f.src and result.dst = f.dst and result.map = f.map
}

define simple_graph_embedding_preimage_set[V, W](f: SimpleGraphEmbedding[V, W], t: Set[W]) -> Set[V] {
    set_preimage(f.map, t)
}

define simple_graph_hom_preimage_set[V, W](f: SimpleGraphHom[V, W], t: Set[W]) -> Set[V] {
    set_preimage(f.map, t)
}

define simple_graph_hom_preimage[V, W](f: SimpleGraphHom[V, W], t: Set[W]) -> SimpleGraph[V] {
    induced_subgraph(f.src, simple_graph_hom_preimage_set(f, t))
}

define simple_graph_iso_image_set[V, W](f: SimpleGraphIso[V, W], s: Set[V]) -> Set[W] {
    set_image(s, f.map)
}

define simple_graph_iso_image[V, W](f: SimpleGraphIso[V, W], s: Set[V]) -> SimpleGraph[W] {
    induced_subgraph(f.dst, simple_graph_iso_image_set(f, s))
}

define simple_graph_iso_preimage_set[V, W](f: SimpleGraphIso[V, W], t: Set[W]) -> Set[V] {
    set_preimage(f.map, t)
}

numerals Nat
theorem six_labels_distinct {
    Nat.0 != Nat.1 and Nat.0 != Nat.2 and Nat.0 != Nat.3 and Nat.0 != Nat.4 and Nat.0 != Nat.5
    and Nat.1 != Nat.2 and Nat.1 != Nat.3 and Nat.1 != Nat.4 and Nat.1 != Nat.5
    and Nat.2 != Nat.3 and Nat.2 != Nat.4 and Nat.2 != Nat.5
    and Nat.3 != Nat.4 and Nat.3 != Nat.5
    and Nat.4 != Nat.5
}

theorem six_vertices_contains(x: Nat) {
    x < Nat.6 implies six_vertices.contains(x)
}

define three_edges_from_zero_share_color(color: (Nat, Nat) -> Bool) -> Bool {
    exists(x: Nat, y: Nat, z: Nat) {
        six_vertices.contains(x) and six_vertices.contains(y) and six_vertices.contains(z)
        and x != Nat.0 and y != Nat.0 and z != Nat.0
        and x != y and x != z and y != z
        and color(Nat.0, x) = color(Nat.0, y) and color(Nat.0, y) = color(Nat.0, z)
    }
}

theorem three_edges_from_zero_share_color_intro(color: (Nat, Nat) -> Bool, x: Nat, y: Nat, z: Nat) {
    six_vertices.contains(x) and six_vertices.contains(y) and six_vertices.contains(z)
    and x != Nat.0 and y != Nat.0 and z != Nat.0
    and x != y and x != z and y != z
    and color(Nat.0, x) = color(Nat.0, y) and color(Nat.0, y) = color(Nat.0, z)
    implies three_edges_from_zero_share_color(color)
}

theorem three_edges_from_zero_share_color_pigeonhole(color: (Nat, Nat) -> Bool) {
    three_edges_from_zero_share_color(color)
}

theorem three_edges_from_zero_share_color_witness(color: (Nat, Nat) -> Bool) {
    three_edges_from_zero_share_color(color) implies exists(x: Nat, y: Nat, z: Nat) {
        six_vertices.contains(x) and six_vertices.contains(y) and six_vertices.contains(z)
        and x != Nat.0 and y != Nat.0 and z != Nat.0
        and x != y and x != z and y != z
        and color(Nat.0, x) = color(Nat.0, y) and color(Nat.0, y) = color(Nat.0, z)
    }
}
