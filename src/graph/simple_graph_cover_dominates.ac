from nat import Nat, lte_trans
from finite_set import FiniteSet, finite_set_subset_refl
from data.finite.finite_set_card import fs_card
from graph.simple_graph import SimpleGraph, simple_graph_adj_comm
from graph.simple_graph_degree import neighborhood, neighborhood_contains_eq
from graph.simple_graph_domination import is_dominating_set, is_dominating_set_intro,
    is_dominated, is_dominated_of_contains, is_dominated_of_adj
from graph.simple_graph_domination_number import domination_number, domination_number_is_least
from graph.simple_graph_vertex_sets import is_vertex_cover, is_vertex_cover_apply

numerals Nat

/// True if every vertex of `s` has a named neighbor inside `s`.
///
/// This is the witnessing form of having no isolated vertices. Stating the witness
/// as a function avoids extracting one from a positive degree, which would need a
/// nonemptiness lemma the finite-set interface does not provide.
define picks_neighbor[V](g: SimpleGraph[V], s: FiniteSet[V], pick: V -> V, v: V) -> Bool {
    s.contains(pick(v)) and g.adj(v, pick(v))
}

/// The chosen vertex lies in the ambient set.
theorem picks_neighbor_contains[V](
    g: SimpleGraph[V], s: FiniteSet[V], pick: V -> V, v: V
) {
    picks_neighbor(g, s, pick, v) implies s.contains(pick(v))
}

/// The chosen vertex is adjacent.
theorem picks_neighbor_adj[V](
    g: SimpleGraph[V], s: FiniteSet[V], pick: V -> V, v: V
) {
    picks_neighbor(g, s, pick, v) implies g.adj(v, pick(v))
}

/// True if `pick` assigns to each vertex of `s` a neighbor lying in `s`.
define has_neighbor_map[V](g: SimpleGraph[V], s: FiniteSet[V], pick: V -> V) -> Bool {
    forall(v: V) {
        s.contains(v) implies picks_neighbor(g, s, pick, v)
    }
}

/// The chosen neighbor of a vertex is a neighbor.
theorem has_neighbor_map_apply[V](
    g: SimpleGraph[V], s: FiniteSet[V], pick: V -> V, v: V
) {
    has_neighbor_map(g, s, pick) and s.contains(v) implies picks_neighbor(g, s, pick, v)
} by {
    if has_neighbor_map(g, s, pick) and s.contains(v) {
        has_neighbor_map(g, s, pick) = forall(u: V) {
            s.contains(u) implies picks_neighbor(g, s, pick, u)
        }
        forall(u: V) {
            s.contains(u) implies picks_neighbor(g, s, pick, u)
        }
        s.contains(v) implies picks_neighbor(g, s, pick, v)
        picks_neighbor(g, s, pick, v)
    }
}

/// A vertex cover of a graph with no isolated vertices is a dominating set.
///
/// A vertex outside the cover has a neighbor, and the edge to that neighbor must
/// meet the cover, so the neighbor is in the cover and dominates the vertex.
theorem vertex_cover_is_dominating[V](
    g: SimpleGraph[V], s: FiniteSet[V], c: FiniteSet[V], pick: V -> V
) {
    has_neighbor_map(g, s, pick) and is_vertex_cover(g, s, c) implies is_dominating_set(g, s, c)
} by {
    if has_neighbor_map(g, s, pick) and is_vertex_cover(g, s, c) {
        forall(v: V) {
            if s.contains(v) {
                if c.contains(v) {
                    is_dominated_of_contains(g, c, v)
                    is_dominated(g, c, v)
                }
                if not c.contains(v) {
                    has_neighbor_map_apply(g, s, pick, v)
                    picks_neighbor(g, s, pick, v)
                    picks_neighbor_contains(g, s, pick, v)
                    s.contains(pick(v))
                    picks_neighbor_adj(g, s, pick, v)
                    g.adj(v, pick(v))
                    is_vertex_cover_apply(g, s, c, v, pick(v))
                    c.contains(v) or c.contains(pick(v))
                    c.contains(pick(v))
                    simple_graph_adj_comm(g, v, pick(v))
                    g.adj(pick(v), v)
                    is_dominated_of_adj(g, c, v, pick(v))
                    is_dominated(g, c, v)
                }
                is_dominated(g, c, v)
            }
            s.contains(v) implies is_dominated(g, c, v)
        }
        is_dominating_set_intro(g, s, c)
        is_dominating_set(g, s, c)
    }
}

/// With no isolated vertices, the domination number is at most the cover number.
///
/// The classical inequality: every vertex cover dominates, so the smallest
/// dominating set is no larger than any cover.
theorem domination_number_le_cover[V](
    g: SimpleGraph[V], s: FiniteSet[V], c: FiniteSet[V], pick: V -> V
) {
    has_neighbor_map(g, s, pick) and c.subset_eq(s) and is_vertex_cover(g, s, c)
        implies domination_number(g, s) <= fs_card(c)
} by {
    if has_neighbor_map(g, s, pick) and c.subset_eq(s) and is_vertex_cover(g, s, c) {
        vertex_cover_is_dominating(g, s, c, pick)
        is_dominating_set(g, s, c)
        domination_number_is_least(g, s, c)
        domination_number(g, s) <= fs_card(c)
    }
}
