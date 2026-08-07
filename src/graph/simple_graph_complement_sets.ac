from nat import Nat
from finite_set import FiniteSet, finite_set_subset_contains
from graph.simple_graph import SimpleGraph, graph_complement, graph_complement_adj_eq,
    simple_graph_adj_ne, simple_graph_adj_irreflexive
from graph.simple_graph_independent import is_independent_in, is_independent_in_apply,
    is_independent_in_intro, is_clique_in, is_clique_in_apply, is_clique_in_intro

numerals Nat

/// An independent set of a graph is a clique of its complement.
///
/// Two distinct members are non-adjacent in the graph, which is exactly what makes
/// them adjacent in the complement.
theorem clique_in_complement_of_independent[V](g: SimpleGraph[V], t: FiniteSet[V]) {
    is_independent_in(g, t) implies is_clique_in(graph_complement(g), t)
} by {
    if is_independent_in(g, t) {
        forall(x: V, y: V) {
            if t.contains(x) and t.contains(y) and x != y {
                is_independent_in_apply(g, t, x, y)
                not g.adj(x, y)
                graph_complement_adj_eq(g, x, y)
                graph_complement(g).adj(x, y) = (x != y and not g.adj(x, y))
                graph_complement(g).adj(x, y)
            }
            t.contains(x) and t.contains(y) and x != y implies graph_complement(g).adj(x, y)
        }
        is_clique_in_intro(graph_complement(g), t)
        is_clique_in(graph_complement(g), t)
    }
}

/// A clique of the complement is an independent set of the graph.
theorem independent_of_clique_in_complement[V](g: SimpleGraph[V], t: FiniteSet[V]) {
    is_clique_in(graph_complement(g), t) implies is_independent_in(g, t)
} by {
    if is_clique_in(graph_complement(g), t) {
        forall(x: V, y: V) {
            if t.contains(x) and t.contains(y) {
                if g.adj(x, y) {
                    simple_graph_adj_ne(g, x, y)
                    x != y
                    is_clique_in_apply(graph_complement(g), t, x, y)
                    graph_complement(g).adj(x, y)
                    graph_complement_adj_eq(g, x, y)
                    graph_complement(g).adj(x, y) = (x != y and not g.adj(x, y))
                    not g.adj(x, y)
                    false
                }
                not g.adj(x, y)
            }
            t.contains(x) and t.contains(y) implies not g.adj(x, y)
        }
        is_independent_in_intro(g, t)
        is_independent_in(g, t)
    }
}

/// A clique of a graph is an independent set of its complement.
theorem independent_in_complement_of_clique[V](g: SimpleGraph[V], t: FiniteSet[V]) {
    is_clique_in(g, t) implies is_independent_in(graph_complement(g), t)
} by {
    if is_clique_in(g, t) {
        forall(x: V, y: V) {
            if t.contains(x) and t.contains(y) {
                if graph_complement(g).adj(x, y) {
                    graph_complement_adj_eq(g, x, y)
                    graph_complement(g).adj(x, y) = (x != y and not g.adj(x, y))
                    x != y
                    not g.adj(x, y)
                    is_clique_in_apply(g, t, x, y)
                    g.adj(x, y)
                    false
                }
                not graph_complement(g).adj(x, y)
            }
            t.contains(x) and t.contains(y) implies not graph_complement(g).adj(x, y)
        }
        is_independent_in_intro(graph_complement(g), t)
        is_independent_in(graph_complement(g), t)
    }
}

/// An independent set of the complement is a clique of the graph.
theorem clique_of_independent_in_complement[V](g: SimpleGraph[V], t: FiniteSet[V]) {
    is_independent_in(graph_complement(g), t) implies is_clique_in(g, t)
} by {
    if is_independent_in(graph_complement(g), t) {
        forall(x: V, y: V) {
            if t.contains(x) and t.contains(y) and x != y {
                is_independent_in_apply(graph_complement(g), t, x, y)
                not graph_complement(g).adj(x, y)
                graph_complement_adj_eq(g, x, y)
                graph_complement(g).adj(x, y) = (x != y and not g.adj(x, y))
                g.adj(x, y)
            }
            t.contains(x) and t.contains(y) and x != y implies g.adj(x, y)
        }
        is_clique_in_intro(g, t)
        is_clique_in(g, t)
    }
}

/// Independence in a graph and the clique property in its complement agree.
theorem independent_iff_clique_in_complement[V](g: SimpleGraph[V], t: FiniteSet[V]) {
    is_independent_in(g, t) = is_clique_in(graph_complement(g), t)
} by {
    if is_independent_in(g, t) {
        clique_in_complement_of_independent(g, t)
        is_clique_in(graph_complement(g), t)
    }
    if is_clique_in(graph_complement(g), t) {
        independent_of_clique_in_complement(g, t)
        is_independent_in(g, t)
    }
}

/// The clique property in a graph and independence in its complement agree.
theorem clique_iff_independent_in_complement[V](g: SimpleGraph[V], t: FiniteSet[V]) {
    is_clique_in(g, t) = is_independent_in(graph_complement(g), t)
} by {
    if is_clique_in(g, t) {
        independent_in_complement_of_clique(g, t)
        is_independent_in(graph_complement(g), t)
    }
    if is_independent_in(graph_complement(g), t) {
        clique_of_independent_in_complement(g, t)
        is_clique_in(g, t)
    }
}
