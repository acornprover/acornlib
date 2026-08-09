from nat import Nat
from finite_set import FiniteSet
from graph.simple_graph import complete_graph
from graph.ramsey import is_edge_2_coloring, has_red_triangle, has_red_triangle_intro,
    has_blue_triangle, has_blue_triangle_intro, has_mono_triangle
from graph.ramsey34_labels import nine_labels_lt_bound
from graph.ramsey34_vertices import nine_vertices, nine_vertices_contains
from graph.ramsey34_good import good_four
from graph.ramsey34_k4 import has_red_k4, has_red_k4_intro, has_blue_k4, has_blue_k4_intro,
    has_mono_k4
from graph.ramsey34_pigeonhole import four_edges_from_zero_share_color,
    four_edges_from_zero_share_color_pigeonhole, four_edges_from_zero_share_color_witness

numerals Nat

/// Ramsey's theorem R(3, 4) <= 9: every red/blue 2-coloring of the edges of the
/// complete graph on nine vertices contains a monochromatic triangle or a
/// monochromatic K4.
///
/// Fix vertex 0. Among its eight incident edges four share a color (pigeonhole);
/// let the witnesses be `w`, `x`, `y`, `z`. If two of the edges among them share
/// the color of the edges to 0, that pair closes a triangle with 0; otherwise the
/// six edges among `w`, `x`, `y`, `z` are all of the other color, which is a K4
/// on its own.
theorem ramsey_r34_upper(color: (Nat, Nat) -> Bool) {
    is_edge_2_coloring(complete_graph[Nat], color) implies
    (has_mono_triangle(color, nine_vertices) or has_mono_k4(color, nine_vertices))
} by {
    if is_edge_2_coloring(complete_graph[Nat], color) {
        four_edges_from_zero_share_color_pigeonhole(color)
        four_edges_from_zero_share_color(color)
        four_edges_from_zero_share_color_witness(color)
        let (w: Nat, x: Nat, y: Nat, z: Nat) satisfy {
            good_four(w, x, y, z)
            and color(Nat.0, w) = color(Nat.0, x) and color(Nat.0, x) = color(Nat.0, y)
            and color(Nat.0, y) = color(Nat.0, z)
        }
        good_four(w, x, y, z) = (w < Nat.9 and x < Nat.9 and y < Nat.9 and z < Nat.9
            and w != Nat.0 and x != Nat.0 and y != Nat.0 and z != Nat.0
            and w != x and w != y and w != z and x != y and x != z and y != z)
        good_four(w, x, y, z)
        w < Nat.9
        x < Nat.9
        y < Nat.9
        z < Nat.9
        nine_vertices_contains(w)
        nine_vertices.contains(w)
        nine_vertices_contains(x)
        nine_vertices.contains(x)
        nine_vertices_contains(y)
        nine_vertices.contains(y)
        nine_vertices_contains(z)
        nine_vertices.contains(z)
        nine_labels_lt_bound
        (Nat.0 < Nat.9 and Nat.1 < Nat.9 and Nat.2 < Nat.9 and Nat.3 < Nat.9 and Nat.4 < Nat.9 and Nat.5 < Nat.9 and Nat.6 < Nat.9 and Nat.7 < Nat.9 and Nat.8 < Nat.9)
        nine_vertices_contains(Nat.0)
        nine_vertices.contains(Nat.0)
        w != Nat.0
        x != Nat.0
        y != Nat.0
        z != Nat.0
        Nat.0 != w
        Nat.0 != x
        Nat.0 != y
        Nat.0 != z
        w != x
        w != y
        w != z
        x != y
        x != z
        y != z
        has_mono_triangle(color, nine_vertices) =
            (has_red_triangle(color, nine_vertices) or has_blue_triangle(color, nine_vertices))
        has_mono_k4(color, nine_vertices) =
            (has_red_k4(color, nine_vertices) or has_blue_k4(color, nine_vertices))
        if color(Nat.0, w) {
            color(Nat.0, w)
            color(Nat.0, x)
            color(Nat.0, y)
            color(Nat.0, z)
            if color(w, x) {
                has_red_triangle_intro(color, nine_vertices, Nat.0, w, x)
                has_red_triangle(color, nine_vertices)
                has_mono_triangle(color, nine_vertices)
                has_mono_triangle(color, nine_vertices) or has_mono_k4(color, nine_vertices)
            } else {
                if color(w, y) {
                    has_red_triangle_intro(color, nine_vertices, Nat.0, w, y)
                    has_red_triangle(color, nine_vertices)
                    has_mono_triangle(color, nine_vertices)
                    has_mono_triangle(color, nine_vertices) or has_mono_k4(color, nine_vertices)
                } else {
                    if color(w, z) {
                        has_red_triangle_intro(color, nine_vertices, Nat.0, w, z)
                        has_red_triangle(color, nine_vertices)
                        has_mono_triangle(color, nine_vertices)
                        has_mono_triangle(color, nine_vertices) or has_mono_k4(color, nine_vertices)
                    } else {
                        if color(x, y) {
                            has_red_triangle_intro(color, nine_vertices, Nat.0, x, y)
                            has_red_triangle(color, nine_vertices)
                            has_mono_triangle(color, nine_vertices)
                            has_mono_triangle(color, nine_vertices) or has_mono_k4(color, nine_vertices)
                        } else {
                            if color(x, z) {
                                has_red_triangle_intro(color, nine_vertices, Nat.0, x, z)
                                has_red_triangle(color, nine_vertices)
                                has_mono_triangle(color, nine_vertices)
                                has_mono_triangle(color, nine_vertices) or has_mono_k4(color, nine_vertices)
                            } else {
                                if color(y, z) {
                                    has_red_triangle_intro(color, nine_vertices, Nat.0, y, z)
                                    has_red_triangle(color, nine_vertices)
                                    has_mono_triangle(color, nine_vertices)
                                    has_mono_triangle(color, nine_vertices) or has_mono_k4(color, nine_vertices)
                                } else {
                                    not color(w, x)
                                    not color(w, y)
                                    not color(w, z)
                                    not color(x, y)
                                    not color(x, z)
                                    not color(y, z)
                                    (nine_vertices.contains(w) and nine_vertices.contains(x)
                                        and nine_vertices.contains(y) and nine_vertices.contains(z))
                                    (w != x and w != y and w != z and x != y and x != z and y != z)
                                    (not color(w, x) and not color(w, y) and not color(w, z)
                                        and not color(x, y) and not color(x, z) and not color(y, z))
                                    has_blue_k4_intro(color, nine_vertices, w, x, y, z)
                                    has_blue_k4(color, nine_vertices)
                                    has_mono_k4(color, nine_vertices)
                                    has_mono_triangle(color, nine_vertices) or has_mono_k4(color, nine_vertices)
                                }
                            }
                        }
                    }
                }
            }
        } else {
            if color(Nat.0, x) {
                color(Nat.0, x) = color(Nat.0, w)
                color(Nat.0, w)
                false
            }
            not color(Nat.0, x)
            if color(Nat.0, y) {
                color(Nat.0, y) = color(Nat.0, x)
                color(Nat.0, x)
                false
            }
            not color(Nat.0, y)
            if color(Nat.0, z) {
                color(Nat.0, z) = color(Nat.0, y)
                color(Nat.0, y)
                false
            }
            not color(Nat.0, z)
            if not color(w, x) {
                has_blue_triangle_intro(color, nine_vertices, Nat.0, w, x)
                has_blue_triangle(color, nine_vertices)
                has_mono_triangle(color, nine_vertices)
                has_mono_triangle(color, nine_vertices) or has_mono_k4(color, nine_vertices)
            } else {
                if not color(w, y) {
                    has_blue_triangle_intro(color, nine_vertices, Nat.0, w, y)
                    has_blue_triangle(color, nine_vertices)
                    has_mono_triangle(color, nine_vertices)
                    has_mono_triangle(color, nine_vertices) or has_mono_k4(color, nine_vertices)
                } else {
                    if not color(w, z) {
                        has_blue_triangle_intro(color, nine_vertices, Nat.0, w, z)
                        has_blue_triangle(color, nine_vertices)
                        has_mono_triangle(color, nine_vertices)
                        has_mono_triangle(color, nine_vertices) or has_mono_k4(color, nine_vertices)
                    } else {
                        if not color(x, y) {
                            has_blue_triangle_intro(color, nine_vertices, Nat.0, x, y)
                            has_blue_triangle(color, nine_vertices)
                            has_mono_triangle(color, nine_vertices)
                            has_mono_triangle(color, nine_vertices) or has_mono_k4(color, nine_vertices)
                        } else {
                            if not color(x, z) {
                                has_blue_triangle_intro(color, nine_vertices, Nat.0, x, z)
                                has_blue_triangle(color, nine_vertices)
                                has_mono_triangle(color, nine_vertices)
                                has_mono_triangle(color, nine_vertices) or has_mono_k4(color, nine_vertices)
                            } else {
                                if not color(y, z) {
                                    has_blue_triangle_intro(color, nine_vertices, Nat.0, y, z)
                                    has_blue_triangle(color, nine_vertices)
                                    has_mono_triangle(color, nine_vertices)
                                    has_mono_triangle(color, nine_vertices) or has_mono_k4(color, nine_vertices)
                                } else {
                                    color(w, x)
                                    color(w, y)
                                    color(w, z)
                                    color(x, y)
                                    color(x, z)
                                    color(y, z)
                                    (nine_vertices.contains(w) and nine_vertices.contains(x)
                                        and nine_vertices.contains(y) and nine_vertices.contains(z))
                                    (w != x and w != y and w != z and x != y and x != z and y != z)
                                    (color(w, x) and color(w, y) and color(w, z)
                                        and color(x, y) and color(x, z) and color(y, z))
                                    has_red_k4_intro(color, nine_vertices, w, x, y, z)
                                    has_red_k4(color, nine_vertices)
                                    has_mono_k4(color, nine_vertices)
                                    has_mono_triangle(color, nine_vertices) or has_mono_k4(color, nine_vertices)
                                }
                            }
                        }
                    }
                }
            }
        }
    }
}

// ============================================================================
// The lower bound R(3, 4) > 8.
//
// The classical 8-vertex coloring witnessing R(3, 4) > 8: the red graph has
// the twelve edges
//
//     01 02 03 14 15 24 26 35 37 47 56 67
//
// which is a 3-regular triangle-free graph on {0, ..., 7} whose complement is
// K4-free (the independence number of the red graph is 3). Equivalently,
//
//     define r34_lb_color(x: Nat, y: Nat) -> Bool {
//         (x = Nat.0 and y = Nat.1) or ... or (y = Nat.6 and x = Nat.7)
//     }
//
// and the statement is
//
//     theorem ramsey_r34_lower_bound {
//         exists(color: (Nat, Nat) -> Bool) {
//             is_edge_2_coloring(complete_graph[Nat], color) and
//             not has_mono_triangle(color, eight_vertices) and
//             not has_mono_k4(color, eight_vertices)
//         }
//     }
//
// Not yet verified: the proof would case-split on the 56 three-vertex subsets
// (no monochromatic triangle) and the 70 four-vertex subsets (no
// monochromatic K4), each a case analysis over the disjuncts of the color
// predicate; proof search does not close those implications reliably. See the
// analogous commented-out treatment of the R(3, 3) lower bound in
// ramsey_lower_bound.ac.
// ============================================================================
