// ============================================================================
// List-index lemmas for the Erdős–Szekeres small case.
//
// The Erdős–Szekeres argument for five distinct numbers (r = s = 3, see
// ramsey_numbers.ac) needs a handful of elementary facts about `get_idx` and
// `value_at`: the zero-th index of a cons list is its head, the successor
// index of a cons list is the index of its tail, every in-bounds position
// carries a value, and a carried value is contained in the list.  The two
// "carries a value" / "contains carried values" properties are proved by
// structural induction, each with a separate cons-step theorem so that the
// induction closes in a small proof-search context.
// ============================================================================

from nat import Nat, zero_or_suc, lt_cancel_suc, lt_add_suc, add_zero_left, suc_sub_one
from list import List

numerals Nat

/// The element at position `i` of the list, if `i` is in bounds.
define value_at(xs: List[Nat], i: Nat, x: Nat) -> Bool {
    xs.get_idx(i) = Option.some(x)
}

/// The zero-th index of a cons list is its head.
theorem get_idx_cons_zero[T](head: T, tail: List[T]) {
    List.cons(head, tail).get_idx(0) = Option.some(head)
} by {
    if 0 > 0 {
        false
    }
    not 0 > 0
    List.cons(head, tail).get_idx(0) = Option.some(head)
}

/// The successor index of a cons list is the index of its tail.
theorem get_idx_cons_suc[T](head: T, tail: List[T], i: Nat) {
    List.cons(head, tail).get_idx(i.suc) = tail.get_idx(i)
} by {
    lt_add_suc(Nat.0, i)
    Nat.0 < Nat.0 + i.suc
    add_zero_left(i.suc)
    Nat.0 + i.suc = i.suc
    Nat.0 < i.suc
    i.suc > 0
    List.cons(head, tail).get_idx(i.suc) = tail.get_idx(i.suc - 1)
    suc_sub_one(i)
    i.suc - 1 = i
    List.cons(head, tail).get_idx(i.suc) = tail.get_idx(i)
}

/// The some-constructor is injective, as a boolean equivalence.
theorem option_some_eq_iff[T](x: T, y: T) {
    (Option.some(x) = Option.some(y)) = (x = y)
} by {
    if Option.some(x) = Option.some(y) {
        some_injective(x, y)
        x = y
    }
    if x = y {
        Option.some(x) = Option.some(y)
    }
    (Option.some(x) = Option.some(y)) = (x = y)
}

/// The value at position zero of a cons list is the head.
theorem value_at_cons_zero(head: Nat, tail: List[Nat], v: Nat) {
    value_at(List.cons(head, tail), 0, v) = (head = v)
} by {
    get_idx_cons_zero(head, tail)
    List.cons(head, tail).get_idx(0) = Option.some(head)
    value_at(List.cons(head, tail), 0, v) = (Option.some(head) = Option.some(v))
    option_some_eq_iff(head, v)
    (Option.some(head) = Option.some(v)) = (head = v)
    value_at(List.cons(head, tail), 0, v) = (head = v)
}

/// The value at a successor position of a cons list is the value in the tail.
theorem value_at_cons_suc(head: Nat, tail: List[Nat], i: Nat, v: Nat) {
    value_at(List.cons(head, tail), i.suc, v) = value_at(tail, i, v)
} by {
    get_idx_cons_suc(head, tail, i)
    List.cons(head, tail).get_idx(i.suc) = tail.get_idx(i)
    value_at(List.cons(head, tail), i.suc, v) = value_at(tail, i, v)
}

/// Every in-bounds position of a list carries a value.
define esz_get_idx_prop(l: List[Nat]) -> Bool {
    forall(idx: Nat) {
        idx < l.length implies exists(x: Nat) {
            l.get_idx(idx) = Option.some(x)
        }
    }
}

/// Prepending a head preserves the property that every in-bounds position
/// carries a value.
theorem esz_get_idx_prop_step(head: Nat, tail: List[Nat]) {
    esz_get_idx_prop(tail) implies esz_get_idx_prop(List.cons(head, tail))
} by {
    if esz_get_idx_prop(tail) {
        forall(idx: Nat) {
            if idx < List.cons(head, tail).length {
                zero_or_suc(idx)
                if idx = Nat.0 {
                    get_idx_cons_zero(head, tail)
                    exists(v: Nat) {
                        List.cons(head, tail).get_idx(idx) = Option.some(v)
                    }
                }
                if idx != Nat.0 {
                    let (j: Nat) satisfy { idx = j.suc }
                    List.cons(head, tail).length = tail.length.suc
                    idx < tail.length.suc
                    j.suc < tail.length.suc
                    lt_cancel_suc(j, tail.length)
                    j < tail.length
                    esz_get_idx_prop(tail) = forall(idx2: Nat) {
                        idx2 < tail.length implies exists(x: Nat) {
                            tail.get_idx(idx2) = Option.some(x)
                        }
                    }
                    forall(idx2: Nat) {
                        idx2 < tail.length implies exists(x: Nat) {
                            tail.get_idx(idx2) = Option.some(x)
                        }
                    }
                    j < tail.length implies exists(x: Nat) {
                        tail.get_idx(j) = Option.some(x)
                    }
                    exists(v: Nat) {
                        tail.get_idx(j) = Option.some(v)
                    }
                    let (v: Nat) satisfy {
                        tail.get_idx(j) = Option.some(v)
                    }
                    get_idx_cons_suc(head, tail, j)
                    List.cons(head, tail).get_idx(idx) = Option.some(v)
                    exists(w2: Nat) {
                        List.cons(head, tail).get_idx(idx) = Option.some(w2)
                    }
                }
            }
        }
        esz_get_idx_prop(List.cons(head, tail)) = forall(idx: Nat) {
            idx < List.cons(head, tail).length implies exists(x: Nat) {
                List.cons(head, tail).get_idx(idx) = Option.some(x)
            }
        }
        forall(idx: Nat) {
            idx < List.cons(head, tail).length implies exists(x: Nat) {
                List.cons(head, tail).get_idx(idx) = Option.some(x)
            }
        }
        esz_get_idx_prop(List.cons(head, tail))
    }
}

/// A position of a list that carries a value witnesses containment.
define esz_value_at_prop(l: List[Nat]) -> Bool {
    forall(idx: Nat, v: Nat) {
        value_at(l, idx, v) implies l.contains(v)
    }
}

/// A value carried by the cons list is contained in it, provided the tail
/// contains its carried values.
theorem esz_value_at_cons_body(head: Nat, tail: List[Nat], idx: Nat, v: Nat) {
    esz_value_at_prop(tail) and value_at(List.cons(head, tail), idx, v)
    implies List.cons(head, tail).contains(v)
} by {
    if esz_value_at_prop(tail) and value_at(List.cons(head, tail), idx, v) {
        if idx = Nat.0 {
            value_at_cons_zero(head, tail, v)
            head = v
            List.cons(head, tail).contains(v)
        }
        if idx != Nat.0 {
            let (j: Nat) satisfy { idx = j.suc }
            value_at_cons_suc(head, tail, j, v)
            value_at(tail, j, v)
            esz_value_at_prop(tail) = forall(idx2: Nat, v2: Nat) {
                value_at(tail, idx2, v2) implies tail.contains(v2)
            }
            forall(idx2: Nat, v2: Nat) {
                value_at(tail, idx2, v2) implies tail.contains(v2)
            }
            value_at(tail, j, v) implies tail.contains(v)
            tail.contains(v)
            List.cons(head, tail).contains(v)
        }
        List.cons(head, tail).contains(v)
    }
}

/// Prepending a head preserves containment of carried values.
theorem esz_value_at_prop_step(head: Nat, tail: List[Nat]) {
    esz_value_at_prop(tail) implies esz_value_at_prop(List.cons(head, tail))
} by {
    if esz_value_at_prop(tail) {
        forall(idx: Nat, v: Nat) {
            value_at(List.cons(head, tail), idx, v) implies List.cons(head, tail).contains(v)
        }
        esz_value_at_prop(List.cons(head, tail)) = forall(idx: Nat, v: Nat) {
            value_at(List.cons(head, tail), idx, v) implies List.cons(head, tail).contains(v)
        }
        esz_value_at_prop(List.cons(head, tail))
    }
}
