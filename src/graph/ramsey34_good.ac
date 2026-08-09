from nat import Nat
from graph.ramsey34_labels import nine_labels_lt_bound, nine_labels_distinct_row0,
    nine_labels_distinct_row1, nine_labels_distinct_row2, nine_labels_distinct_row3,
    nine_labels_distinct_row4, nine_labels_distinct_row5, nine_labels_distinct_row6,
    nine_labels_distinct_row7

numerals Nat

/// The label 1 is below nine.
theorem lt_1_9 {
    Nat.1 < Nat.9
} by {
    nine_labels_lt_bound
    Nat.1 < Nat.9
}


/// The label 2 is below nine.
theorem lt_2_9 {
    Nat.2 < Nat.9
} by {
    nine_labels_lt_bound
    Nat.2 < Nat.9
}


/// The label 3 is below nine.
theorem lt_3_9 {
    Nat.3 < Nat.9
} by {
    nine_labels_lt_bound
    Nat.3 < Nat.9
}


/// The label 4 is below nine.
theorem lt_4_9 {
    Nat.4 < Nat.9
} by {
    nine_labels_lt_bound
    Nat.4 < Nat.9
}


/// The label 5 is below nine.
theorem lt_5_9 {
    Nat.5 < Nat.9
} by {
    nine_labels_lt_bound
    Nat.5 < Nat.9
}


/// The label 6 is below nine.
theorem lt_6_9 {
    Nat.6 < Nat.9
} by {
    nine_labels_lt_bound
    Nat.6 < Nat.9
}


/// The label 7 is below nine.
theorem lt_7_9 {
    Nat.7 < Nat.9
} by {
    nine_labels_lt_bound
    Nat.7 < Nat.9
}


/// The label 8 is below nine.
theorem lt_8_9 {
    Nat.8 < Nat.9
} by {
    nine_labels_lt_bound
    Nat.8 < Nat.9
}


/// The labels 1 and 0 are distinct.
theorem ne_1_0 {
    Nat.1 != Nat.0
} by {
    nine_labels_distinct_row0
    Nat.1 != Nat.0
}


/// The labels 2 and 0 are distinct.
theorem ne_2_0 {
    Nat.2 != Nat.0
} by {
    nine_labels_distinct_row0
    Nat.2 != Nat.0
}


/// The labels 3 and 0 are distinct.
theorem ne_3_0 {
    Nat.3 != Nat.0
} by {
    nine_labels_distinct_row0
    Nat.3 != Nat.0
}


/// The labels 4 and 0 are distinct.
theorem ne_4_0 {
    Nat.4 != Nat.0
} by {
    nine_labels_distinct_row0
    Nat.4 != Nat.0
}


/// The labels 5 and 0 are distinct.
theorem ne_5_0 {
    Nat.5 != Nat.0
} by {
    nine_labels_distinct_row0
    Nat.5 != Nat.0
}


/// The labels 6 and 0 are distinct.
theorem ne_6_0 {
    Nat.6 != Nat.0
} by {
    nine_labels_distinct_row0
    Nat.6 != Nat.0
}


/// The labels 7 and 0 are distinct.
theorem ne_7_0 {
    Nat.7 != Nat.0
} by {
    nine_labels_distinct_row0
    Nat.7 != Nat.0
}


/// The labels 8 and 0 are distinct.
theorem ne_8_0 {
    Nat.8 != Nat.0
} by {
    nine_labels_distinct_row0
    Nat.8 != Nat.0
}


/// The labels 1 and 2 are distinct.
theorem ne_1_2 {
    Nat.1 != Nat.2
} by {
    nine_labels_distinct_row1
    Nat.1 != Nat.2
}


/// The labels 1 and 3 are distinct.
theorem ne_1_3 {
    Nat.1 != Nat.3
} by {
    nine_labels_distinct_row1
    Nat.1 != Nat.3
}


/// The labels 1 and 4 are distinct.
theorem ne_1_4 {
    Nat.1 != Nat.4
} by {
    nine_labels_distinct_row1
    Nat.1 != Nat.4
}


/// The labels 1 and 5 are distinct.
theorem ne_1_5 {
    Nat.1 != Nat.5
} by {
    nine_labels_distinct_row1
    Nat.1 != Nat.5
}


/// The labels 1 and 6 are distinct.
theorem ne_1_6 {
    Nat.1 != Nat.6
} by {
    nine_labels_distinct_row1
    Nat.1 != Nat.6
}


/// The labels 1 and 7 are distinct.
theorem ne_1_7 {
    Nat.1 != Nat.7
} by {
    nine_labels_distinct_row1
    Nat.1 != Nat.7
}


/// The labels 1 and 8 are distinct.
theorem ne_1_8 {
    Nat.1 != Nat.8
} by {
    nine_labels_distinct_row1
    Nat.1 != Nat.8
}


/// The labels 2 and 3 are distinct.
theorem ne_2_3 {
    Nat.2 != Nat.3
} by {
    nine_labels_distinct_row2
    Nat.2 != Nat.3
}


/// The labels 2 and 4 are distinct.
theorem ne_2_4 {
    Nat.2 != Nat.4
} by {
    nine_labels_distinct_row2
    Nat.2 != Nat.4
}


/// The labels 2 and 5 are distinct.
theorem ne_2_5 {
    Nat.2 != Nat.5
} by {
    nine_labels_distinct_row2
    Nat.2 != Nat.5
}


/// The labels 2 and 6 are distinct.
theorem ne_2_6 {
    Nat.2 != Nat.6
} by {
    nine_labels_distinct_row2
    Nat.2 != Nat.6
}


/// The labels 2 and 7 are distinct.
theorem ne_2_7 {
    Nat.2 != Nat.7
} by {
    nine_labels_distinct_row2
    Nat.2 != Nat.7
}


/// The labels 2 and 8 are distinct.
theorem ne_2_8 {
    Nat.2 != Nat.8
} by {
    nine_labels_distinct_row2
    Nat.2 != Nat.8
}


/// The labels 3 and 4 are distinct.
theorem ne_3_4 {
    Nat.3 != Nat.4
} by {
    nine_labels_distinct_row3
    Nat.3 != Nat.4
}


/// The labels 3 and 5 are distinct.
theorem ne_3_5 {
    Nat.3 != Nat.5
} by {
    nine_labels_distinct_row3
    Nat.3 != Nat.5
}


/// The labels 3 and 6 are distinct.
theorem ne_3_6 {
    Nat.3 != Nat.6
} by {
    nine_labels_distinct_row3
    Nat.3 != Nat.6
}


/// The labels 3 and 7 are distinct.
theorem ne_3_7 {
    Nat.3 != Nat.7
} by {
    nine_labels_distinct_row3
    Nat.3 != Nat.7
}


/// The labels 3 and 8 are distinct.
theorem ne_3_8 {
    Nat.3 != Nat.8
} by {
    nine_labels_distinct_row3
    Nat.3 != Nat.8
}


/// The labels 4 and 5 are distinct.
theorem ne_4_5 {
    Nat.4 != Nat.5
} by {
    nine_labels_distinct_row4
    Nat.4 != Nat.5
}


/// The labels 4 and 6 are distinct.
theorem ne_4_6 {
    Nat.4 != Nat.6
} by {
    nine_labels_distinct_row4
    Nat.4 != Nat.6
}


/// The labels 4 and 7 are distinct.
theorem ne_4_7 {
    Nat.4 != Nat.7
} by {
    nine_labels_distinct_row4
    Nat.4 != Nat.7
}


/// The labels 4 and 8 are distinct.
theorem ne_4_8 {
    Nat.4 != Nat.8
} by {
    nine_labels_distinct_row4
    Nat.4 != Nat.8
}


/// The labels 5 and 6 are distinct.
theorem ne_5_6 {
    Nat.5 != Nat.6
} by {
    nine_labels_distinct_row5
    Nat.5 != Nat.6
}


/// The labels 5 and 7 are distinct.
theorem ne_5_7 {
    Nat.5 != Nat.7
} by {
    nine_labels_distinct_row5
    Nat.5 != Nat.7
}


/// The labels 5 and 8 are distinct.
theorem ne_5_8 {
    Nat.5 != Nat.8
} by {
    nine_labels_distinct_row5
    Nat.5 != Nat.8
}


/// The labels 6 and 7 are distinct.
theorem ne_6_7 {
    Nat.6 != Nat.7
} by {
    nine_labels_distinct_row6
    Nat.6 != Nat.7
}


/// The labels 6 and 8 are distinct.
theorem ne_6_8 {
    Nat.6 != Nat.8
} by {
    nine_labels_distinct_row6
    Nat.6 != Nat.8
}


/// The labels 7 and 8 are distinct.
theorem ne_7_8 {
    Nat.7 != Nat.8
} by {
    nine_labels_distinct_row7
    Nat.7 != Nat.8
}


/// The four vertex-quality facts of a pigeonhole witness, in arithmetic form.
///
/// The witnesses are naturals below nine, different from zero, and pairwise
/// distinct, which is everything the K4 argument needs to know about them.
define good_four(w: Nat, x: Nat, y: Nat, z: Nat) -> Bool {
    w < Nat.9 and x < Nat.9 and y < Nat.9 and z < Nat.9
    and w != Nat.0 and x != Nat.0 and y != Nat.0 and z != Nat.0
    and w != x and w != y and w != z and x != y and x != z and y != z
}

/// The pointwise vertex-quality facts package into `good_four`.
theorem good_four_intro(w: Nat, x: Nat, y: Nat, z: Nat) {
    w < Nat.9 and x < Nat.9 and y < Nat.9 and z < Nat.9
    and w != Nat.0 and x != Nat.0 and y != Nat.0 and z != Nat.0
    and w != x and w != y and w != z and x != y and x != z and y != z
    implies good_four(w, x, y, z)
} by {
    if w < Nat.9 and x < Nat.9 and y < Nat.9 and z < Nat.9
        and w != Nat.0 and x != Nat.0 and y != Nat.0 and z != Nat.0
        and w != x and w != y and w != z and x != y and x != z and y != z {
        good_four(w, x, y, z) = (w < Nat.9 and x < Nat.9 and y < Nat.9 and z < Nat.9
            and w != Nat.0 and x != Nat.0 and y != Nat.0 and z != Nat.0
            and w != x and w != y and w != z and x != y and x != z and y != z)
        good_four(w, x, y, z)
    }
}

/// The good vertices 1, 2, 3, 4 of the nine-element set are a good four.
theorem good_four_1234 {
    good_four(Nat.1, Nat.2, Nat.3, Nat.4)
} by {
    lt_1_9
    lt_2_9
    lt_3_9
    lt_4_9
    ne_1_0
    ne_2_0
    ne_3_0
    ne_4_0
    ne_1_2
    ne_1_3
    ne_1_4
    ne_2_3
    ne_2_4
    ne_3_4
    good_four_intro(Nat.1, Nat.2, Nat.3, Nat.4)
    good_four(Nat.1, Nat.2, Nat.3, Nat.4)
}

/// The good vertices 1, 2, 3, 5 of the nine-element set are a good four.
theorem good_four_1235 {
    good_four(Nat.1, Nat.2, Nat.3, Nat.5)
} by {
    lt_1_9
    lt_2_9
    lt_3_9
    lt_5_9
    ne_1_0
    ne_2_0
    ne_3_0
    ne_5_0
    ne_1_2
    ne_1_3
    ne_1_5
    ne_2_3
    ne_2_5
    ne_3_5
    good_four_intro(Nat.1, Nat.2, Nat.3, Nat.5)
    good_four(Nat.1, Nat.2, Nat.3, Nat.5)
}

/// The good vertices 1, 2, 3, 6 of the nine-element set are a good four.
theorem good_four_1236 {
    good_four(Nat.1, Nat.2, Nat.3, Nat.6)
} by {
    lt_1_9
    lt_2_9
    lt_3_9
    lt_6_9
    ne_1_0
    ne_2_0
    ne_3_0
    ne_6_0
    ne_1_2
    ne_1_3
    ne_1_6
    ne_2_3
    ne_2_6
    ne_3_6
    good_four_intro(Nat.1, Nat.2, Nat.3, Nat.6)
    good_four(Nat.1, Nat.2, Nat.3, Nat.6)
}

/// The good vertices 1, 2, 3, 7 of the nine-element set are a good four.
theorem good_four_1237 {
    good_four(Nat.1, Nat.2, Nat.3, Nat.7)
} by {
    lt_1_9
    lt_2_9
    lt_3_9
    lt_7_9
    ne_1_0
    ne_2_0
    ne_3_0
    ne_7_0
    ne_1_2
    ne_1_3
    ne_1_7
    ne_2_3
    ne_2_7
    ne_3_7
    good_four_intro(Nat.1, Nat.2, Nat.3, Nat.7)
    good_four(Nat.1, Nat.2, Nat.3, Nat.7)
}

/// The good vertices 1, 2, 3, 8 of the nine-element set are a good four.
theorem good_four_1238 {
    good_four(Nat.1, Nat.2, Nat.3, Nat.8)
} by {
    lt_1_9
    lt_2_9
    lt_3_9
    lt_8_9
    ne_1_0
    ne_2_0
    ne_3_0
    ne_8_0
    ne_1_2
    ne_1_3
    ne_1_8
    ne_2_3
    ne_2_8
    ne_3_8
    good_four_intro(Nat.1, Nat.2, Nat.3, Nat.8)
    good_four(Nat.1, Nat.2, Nat.3, Nat.8)
}

/// The good vertices 1, 2, 4, 5 of the nine-element set are a good four.
theorem good_four_1245 {
    good_four(Nat.1, Nat.2, Nat.4, Nat.5)
} by {
    lt_1_9
    lt_2_9
    lt_4_9
    lt_5_9
    ne_1_0
    ne_2_0
    ne_4_0
    ne_5_0
    ne_1_2
    ne_1_4
    ne_1_5
    ne_2_4
    ne_2_5
    ne_4_5
    good_four_intro(Nat.1, Nat.2, Nat.4, Nat.5)
    good_four(Nat.1, Nat.2, Nat.4, Nat.5)
}

/// The good vertices 1, 2, 4, 6 of the nine-element set are a good four.
theorem good_four_1246 {
    good_four(Nat.1, Nat.2, Nat.4, Nat.6)
} by {
    lt_1_9
    lt_2_9
    lt_4_9
    lt_6_9
    ne_1_0
    ne_2_0
    ne_4_0
    ne_6_0
    ne_1_2
    ne_1_4
    ne_1_6
    ne_2_4
    ne_2_6
    ne_4_6
    good_four_intro(Nat.1, Nat.2, Nat.4, Nat.6)
    good_four(Nat.1, Nat.2, Nat.4, Nat.6)
}

/// The good vertices 1, 2, 4, 7 of the nine-element set are a good four.
theorem good_four_1247 {
    good_four(Nat.1, Nat.2, Nat.4, Nat.7)
} by {
    lt_1_9
    lt_2_9
    lt_4_9
    lt_7_9
    ne_1_0
    ne_2_0
    ne_4_0
    ne_7_0
    ne_1_2
    ne_1_4
    ne_1_7
    ne_2_4
    ne_2_7
    ne_4_7
    good_four_intro(Nat.1, Nat.2, Nat.4, Nat.7)
    good_four(Nat.1, Nat.2, Nat.4, Nat.7)
}

/// The good vertices 1, 2, 4, 8 of the nine-element set are a good four.
theorem good_four_1248 {
    good_four(Nat.1, Nat.2, Nat.4, Nat.8)
} by {
    lt_1_9
    lt_2_9
    lt_4_9
    lt_8_9
    ne_1_0
    ne_2_0
    ne_4_0
    ne_8_0
    ne_1_2
    ne_1_4
    ne_1_8
    ne_2_4
    ne_2_8
    ne_4_8
    good_four_intro(Nat.1, Nat.2, Nat.4, Nat.8)
    good_four(Nat.1, Nat.2, Nat.4, Nat.8)
}

/// The good vertices 1, 2, 5, 6 of the nine-element set are a good four.
theorem good_four_1256 {
    good_four(Nat.1, Nat.2, Nat.5, Nat.6)
} by {
    lt_1_9
    lt_2_9
    lt_5_9
    lt_6_9
    ne_1_0
    ne_2_0
    ne_5_0
    ne_6_0
    ne_1_2
    ne_1_5
    ne_1_6
    ne_2_5
    ne_2_6
    ne_5_6
    good_four_intro(Nat.1, Nat.2, Nat.5, Nat.6)
    good_four(Nat.1, Nat.2, Nat.5, Nat.6)
}

/// The good vertices 1, 2, 5, 7 of the nine-element set are a good four.
theorem good_four_1257 {
    good_four(Nat.1, Nat.2, Nat.5, Nat.7)
} by {
    lt_1_9
    lt_2_9
    lt_5_9
    lt_7_9
    ne_1_0
    ne_2_0
    ne_5_0
    ne_7_0
    ne_1_2
    ne_1_5
    ne_1_7
    ne_2_5
    ne_2_7
    ne_5_7
    good_four_intro(Nat.1, Nat.2, Nat.5, Nat.7)
    good_four(Nat.1, Nat.2, Nat.5, Nat.7)
}

/// The good vertices 1, 2, 5, 8 of the nine-element set are a good four.
theorem good_four_1258 {
    good_four(Nat.1, Nat.2, Nat.5, Nat.8)
} by {
    lt_1_9
    lt_2_9
    lt_5_9
    lt_8_9
    ne_1_0
    ne_2_0
    ne_5_0
    ne_8_0
    ne_1_2
    ne_1_5
    ne_1_8
    ne_2_5
    ne_2_8
    ne_5_8
    good_four_intro(Nat.1, Nat.2, Nat.5, Nat.8)
    good_four(Nat.1, Nat.2, Nat.5, Nat.8)
}

/// The good vertices 1, 2, 6, 7 of the nine-element set are a good four.
theorem good_four_1267 {
    good_four(Nat.1, Nat.2, Nat.6, Nat.7)
} by {
    lt_1_9
    lt_2_9
    lt_6_9
    lt_7_9
    ne_1_0
    ne_2_0
    ne_6_0
    ne_7_0
    ne_1_2
    ne_1_6
    ne_1_7
    ne_2_6
    ne_2_7
    ne_6_7
    good_four_intro(Nat.1, Nat.2, Nat.6, Nat.7)
    good_four(Nat.1, Nat.2, Nat.6, Nat.7)
}

/// The good vertices 1, 2, 6, 8 of the nine-element set are a good four.
theorem good_four_1268 {
    good_four(Nat.1, Nat.2, Nat.6, Nat.8)
} by {
    lt_1_9
    lt_2_9
    lt_6_9
    lt_8_9
    ne_1_0
    ne_2_0
    ne_6_0
    ne_8_0
    ne_1_2
    ne_1_6
    ne_1_8
    ne_2_6
    ne_2_8
    ne_6_8
    good_four_intro(Nat.1, Nat.2, Nat.6, Nat.8)
    good_four(Nat.1, Nat.2, Nat.6, Nat.8)
}

/// The good vertices 1, 2, 7, 8 of the nine-element set are a good four.
theorem good_four_1278 {
    good_four(Nat.1, Nat.2, Nat.7, Nat.8)
} by {
    lt_1_9
    lt_2_9
    lt_7_9
    lt_8_9
    ne_1_0
    ne_2_0
    ne_7_0
    ne_8_0
    ne_1_2
    ne_1_7
    ne_1_8
    ne_2_7
    ne_2_8
    ne_7_8
    good_four_intro(Nat.1, Nat.2, Nat.7, Nat.8)
    good_four(Nat.1, Nat.2, Nat.7, Nat.8)
}

/// The good vertices 1, 3, 4, 5 of the nine-element set are a good four.
theorem good_four_1345 {
    good_four(Nat.1, Nat.3, Nat.4, Nat.5)
} by {
    lt_1_9
    lt_3_9
    lt_4_9
    lt_5_9
    ne_1_0
    ne_3_0
    ne_4_0
    ne_5_0
    ne_1_3
    ne_1_4
    ne_1_5
    ne_3_4
    ne_3_5
    ne_4_5
    good_four_intro(Nat.1, Nat.3, Nat.4, Nat.5)
    good_four(Nat.1, Nat.3, Nat.4, Nat.5)
}

/// The good vertices 1, 3, 4, 6 of the nine-element set are a good four.
theorem good_four_1346 {
    good_four(Nat.1, Nat.3, Nat.4, Nat.6)
} by {
    lt_1_9
    lt_3_9
    lt_4_9
    lt_6_9
    ne_1_0
    ne_3_0
    ne_4_0
    ne_6_0
    ne_1_3
    ne_1_4
    ne_1_6
    ne_3_4
    ne_3_6
    ne_4_6
    good_four_intro(Nat.1, Nat.3, Nat.4, Nat.6)
    good_four(Nat.1, Nat.3, Nat.4, Nat.6)
}

/// The good vertices 1, 3, 4, 7 of the nine-element set are a good four.
theorem good_four_1347 {
    good_four(Nat.1, Nat.3, Nat.4, Nat.7)
} by {
    lt_1_9
    lt_3_9
    lt_4_9
    lt_7_9
    ne_1_0
    ne_3_0
    ne_4_0
    ne_7_0
    ne_1_3
    ne_1_4
    ne_1_7
    ne_3_4
    ne_3_7
    ne_4_7
    good_four_intro(Nat.1, Nat.3, Nat.4, Nat.7)
    good_four(Nat.1, Nat.3, Nat.4, Nat.7)
}

/// The good vertices 1, 3, 4, 8 of the nine-element set are a good four.
theorem good_four_1348 {
    good_four(Nat.1, Nat.3, Nat.4, Nat.8)
} by {
    lt_1_9
    lt_3_9
    lt_4_9
    lt_8_9
    ne_1_0
    ne_3_0
    ne_4_0
    ne_8_0
    ne_1_3
    ne_1_4
    ne_1_8
    ne_3_4
    ne_3_8
    ne_4_8
    good_four_intro(Nat.1, Nat.3, Nat.4, Nat.8)
    good_four(Nat.1, Nat.3, Nat.4, Nat.8)
}

/// The good vertices 1, 3, 5, 6 of the nine-element set are a good four.
theorem good_four_1356 {
    good_four(Nat.1, Nat.3, Nat.5, Nat.6)
} by {
    lt_1_9
    lt_3_9
    lt_5_9
    lt_6_9
    ne_1_0
    ne_3_0
    ne_5_0
    ne_6_0
    ne_1_3
    ne_1_5
    ne_1_6
    ne_3_5
    ne_3_6
    ne_5_6
    good_four_intro(Nat.1, Nat.3, Nat.5, Nat.6)
    good_four(Nat.1, Nat.3, Nat.5, Nat.6)
}

/// The good vertices 1, 3, 5, 7 of the nine-element set are a good four.
theorem good_four_1357 {
    good_four(Nat.1, Nat.3, Nat.5, Nat.7)
} by {
    lt_1_9
    lt_3_9
    lt_5_9
    lt_7_9
    ne_1_0
    ne_3_0
    ne_5_0
    ne_7_0
    ne_1_3
    ne_1_5
    ne_1_7
    ne_3_5
    ne_3_7
    ne_5_7
    good_four_intro(Nat.1, Nat.3, Nat.5, Nat.7)
    good_four(Nat.1, Nat.3, Nat.5, Nat.7)
}

/// The good vertices 1, 3, 5, 8 of the nine-element set are a good four.
theorem good_four_1358 {
    good_four(Nat.1, Nat.3, Nat.5, Nat.8)
} by {
    lt_1_9
    lt_3_9
    lt_5_9
    lt_8_9
    ne_1_0
    ne_3_0
    ne_5_0
    ne_8_0
    ne_1_3
    ne_1_5
    ne_1_8
    ne_3_5
    ne_3_8
    ne_5_8
    good_four_intro(Nat.1, Nat.3, Nat.5, Nat.8)
    good_four(Nat.1, Nat.3, Nat.5, Nat.8)
}

/// The good vertices 1, 3, 6, 7 of the nine-element set are a good four.
theorem good_four_1367 {
    good_four(Nat.1, Nat.3, Nat.6, Nat.7)
} by {
    lt_1_9
    lt_3_9
    lt_6_9
    lt_7_9
    ne_1_0
    ne_3_0
    ne_6_0
    ne_7_0
    ne_1_3
    ne_1_6
    ne_1_7
    ne_3_6
    ne_3_7
    ne_6_7
    good_four_intro(Nat.1, Nat.3, Nat.6, Nat.7)
    good_four(Nat.1, Nat.3, Nat.6, Nat.7)
}

/// The good vertices 1, 3, 6, 8 of the nine-element set are a good four.
theorem good_four_1368 {
    good_four(Nat.1, Nat.3, Nat.6, Nat.8)
} by {
    lt_1_9
    lt_3_9
    lt_6_9
    lt_8_9
    ne_1_0
    ne_3_0
    ne_6_0
    ne_8_0
    ne_1_3
    ne_1_6
    ne_1_8
    ne_3_6
    ne_3_8
    ne_6_8
    good_four_intro(Nat.1, Nat.3, Nat.6, Nat.8)
    good_four(Nat.1, Nat.3, Nat.6, Nat.8)
}

/// The good vertices 1, 3, 7, 8 of the nine-element set are a good four.
theorem good_four_1378 {
    good_four(Nat.1, Nat.3, Nat.7, Nat.8)
} by {
    lt_1_9
    lt_3_9
    lt_7_9
    lt_8_9
    ne_1_0
    ne_3_0
    ne_7_0
    ne_8_0
    ne_1_3
    ne_1_7
    ne_1_8
    ne_3_7
    ne_3_8
    ne_7_8
    good_four_intro(Nat.1, Nat.3, Nat.7, Nat.8)
    good_four(Nat.1, Nat.3, Nat.7, Nat.8)
}

/// The good vertices 1, 4, 5, 6 of the nine-element set are a good four.
theorem good_four_1456 {
    good_four(Nat.1, Nat.4, Nat.5, Nat.6)
} by {
    lt_1_9
    lt_4_9
    lt_5_9
    lt_6_9
    ne_1_0
    ne_4_0
    ne_5_0
    ne_6_0
    ne_1_4
    ne_1_5
    ne_1_6
    ne_4_5
    ne_4_6
    ne_5_6
    good_four_intro(Nat.1, Nat.4, Nat.5, Nat.6)
    good_four(Nat.1, Nat.4, Nat.5, Nat.6)
}

/// The good vertices 1, 4, 5, 7 of the nine-element set are a good four.
theorem good_four_1457 {
    good_four(Nat.1, Nat.4, Nat.5, Nat.7)
} by {
    lt_1_9
    lt_4_9
    lt_5_9
    lt_7_9
    ne_1_0
    ne_4_0
    ne_5_0
    ne_7_0
    ne_1_4
    ne_1_5
    ne_1_7
    ne_4_5
    ne_4_7
    ne_5_7
    good_four_intro(Nat.1, Nat.4, Nat.5, Nat.7)
    good_four(Nat.1, Nat.4, Nat.5, Nat.7)
}

/// The good vertices 1, 4, 5, 8 of the nine-element set are a good four.
theorem good_four_1458 {
    good_four(Nat.1, Nat.4, Nat.5, Nat.8)
} by {
    lt_1_9
    lt_4_9
    lt_5_9
    lt_8_9
    ne_1_0
    ne_4_0
    ne_5_0
    ne_8_0
    ne_1_4
    ne_1_5
    ne_1_8
    ne_4_5
    ne_4_8
    ne_5_8
    good_four_intro(Nat.1, Nat.4, Nat.5, Nat.8)
    good_four(Nat.1, Nat.4, Nat.5, Nat.8)
}

/// The good vertices 1, 4, 6, 7 of the nine-element set are a good four.
theorem good_four_1467 {
    good_four(Nat.1, Nat.4, Nat.6, Nat.7)
} by {
    lt_1_9
    lt_4_9
    lt_6_9
    lt_7_9
    ne_1_0
    ne_4_0
    ne_6_0
    ne_7_0
    ne_1_4
    ne_1_6
    ne_1_7
    ne_4_6
    ne_4_7
    ne_6_7
    good_four_intro(Nat.1, Nat.4, Nat.6, Nat.7)
    good_four(Nat.1, Nat.4, Nat.6, Nat.7)
}

/// The good vertices 1, 4, 6, 8 of the nine-element set are a good four.
theorem good_four_1468 {
    good_four(Nat.1, Nat.4, Nat.6, Nat.8)
} by {
    lt_1_9
    lt_4_9
    lt_6_9
    lt_8_9
    ne_1_0
    ne_4_0
    ne_6_0
    ne_8_0
    ne_1_4
    ne_1_6
    ne_1_8
    ne_4_6
    ne_4_8
    ne_6_8
    good_four_intro(Nat.1, Nat.4, Nat.6, Nat.8)
    good_four(Nat.1, Nat.4, Nat.6, Nat.8)
}

/// The good vertices 1, 4, 7, 8 of the nine-element set are a good four.
theorem good_four_1478 {
    good_four(Nat.1, Nat.4, Nat.7, Nat.8)
} by {
    lt_1_9
    lt_4_9
    lt_7_9
    lt_8_9
    ne_1_0
    ne_4_0
    ne_7_0
    ne_8_0
    ne_1_4
    ne_1_7
    ne_1_8
    ne_4_7
    ne_4_8
    ne_7_8
    good_four_intro(Nat.1, Nat.4, Nat.7, Nat.8)
    good_four(Nat.1, Nat.4, Nat.7, Nat.8)
}

/// The good vertices 1, 5, 6, 7 of the nine-element set are a good four.
theorem good_four_1567 {
    good_four(Nat.1, Nat.5, Nat.6, Nat.7)
} by {
    lt_1_9
    lt_5_9
    lt_6_9
    lt_7_9
    ne_1_0
    ne_5_0
    ne_6_0
    ne_7_0
    ne_1_5
    ne_1_6
    ne_1_7
    ne_5_6
    ne_5_7
    ne_6_7
    good_four_intro(Nat.1, Nat.5, Nat.6, Nat.7)
    good_four(Nat.1, Nat.5, Nat.6, Nat.7)
}

/// The good vertices 1, 5, 6, 8 of the nine-element set are a good four.
theorem good_four_1568 {
    good_four(Nat.1, Nat.5, Nat.6, Nat.8)
} by {
    lt_1_9
    lt_5_9
    lt_6_9
    lt_8_9
    ne_1_0
    ne_5_0
    ne_6_0
    ne_8_0
    ne_1_5
    ne_1_6
    ne_1_8
    ne_5_6
    ne_5_8
    ne_6_8
    good_four_intro(Nat.1, Nat.5, Nat.6, Nat.8)
    good_four(Nat.1, Nat.5, Nat.6, Nat.8)
}

/// The good vertices 1, 5, 7, 8 of the nine-element set are a good four.
theorem good_four_1578 {
    good_four(Nat.1, Nat.5, Nat.7, Nat.8)
} by {
    lt_1_9
    lt_5_9
    lt_7_9
    lt_8_9
    ne_1_0
    ne_5_0
    ne_7_0
    ne_8_0
    ne_1_5
    ne_1_7
    ne_1_8
    ne_5_7
    ne_5_8
    ne_7_8
    good_four_intro(Nat.1, Nat.5, Nat.7, Nat.8)
    good_four(Nat.1, Nat.5, Nat.7, Nat.8)
}

/// The good vertices 1, 6, 7, 8 of the nine-element set are a good four.
theorem good_four_1678 {
    good_four(Nat.1, Nat.6, Nat.7, Nat.8)
} by {
    lt_1_9
    lt_6_9
    lt_7_9
    lt_8_9
    ne_1_0
    ne_6_0
    ne_7_0
    ne_8_0
    ne_1_6
    ne_1_7
    ne_1_8
    ne_6_7
    ne_6_8
    ne_7_8
    good_four_intro(Nat.1, Nat.6, Nat.7, Nat.8)
    good_four(Nat.1, Nat.6, Nat.7, Nat.8)
}

/// The good vertices 2, 3, 4, 5 of the nine-element set are a good four.
theorem good_four_2345 {
    good_four(Nat.2, Nat.3, Nat.4, Nat.5)
} by {
    lt_2_9
    lt_3_9
    lt_4_9
    lt_5_9
    ne_2_0
    ne_3_0
    ne_4_0
    ne_5_0
    ne_2_3
    ne_2_4
    ne_2_5
    ne_3_4
    ne_3_5
    ne_4_5
    good_four_intro(Nat.2, Nat.3, Nat.4, Nat.5)
    good_four(Nat.2, Nat.3, Nat.4, Nat.5)
}

/// The good vertices 2, 3, 4, 6 of the nine-element set are a good four.
theorem good_four_2346 {
    good_four(Nat.2, Nat.3, Nat.4, Nat.6)
} by {
    lt_2_9
    lt_3_9
    lt_4_9
    lt_6_9
    ne_2_0
    ne_3_0
    ne_4_0
    ne_6_0
    ne_2_3
    ne_2_4
    ne_2_6
    ne_3_4
    ne_3_6
    ne_4_6
    good_four_intro(Nat.2, Nat.3, Nat.4, Nat.6)
    good_four(Nat.2, Nat.3, Nat.4, Nat.6)
}

/// The good vertices 2, 3, 4, 7 of the nine-element set are a good four.
theorem good_four_2347 {
    good_four(Nat.2, Nat.3, Nat.4, Nat.7)
} by {
    lt_2_9
    lt_3_9
    lt_4_9
    lt_7_9
    ne_2_0
    ne_3_0
    ne_4_0
    ne_7_0
    ne_2_3
    ne_2_4
    ne_2_7
    ne_3_4
    ne_3_7
    ne_4_7
    good_four_intro(Nat.2, Nat.3, Nat.4, Nat.7)
    good_four(Nat.2, Nat.3, Nat.4, Nat.7)
}

/// The good vertices 2, 3, 4, 8 of the nine-element set are a good four.
theorem good_four_2348 {
    good_four(Nat.2, Nat.3, Nat.4, Nat.8)
} by {
    lt_2_9
    lt_3_9
    lt_4_9
    lt_8_9
    ne_2_0
    ne_3_0
    ne_4_0
    ne_8_0
    ne_2_3
    ne_2_4
    ne_2_8
    ne_3_4
    ne_3_8
    ne_4_8
    good_four_intro(Nat.2, Nat.3, Nat.4, Nat.8)
    good_four(Nat.2, Nat.3, Nat.4, Nat.8)
}

/// The good vertices 2, 3, 5, 6 of the nine-element set are a good four.
theorem good_four_2356 {
    good_four(Nat.2, Nat.3, Nat.5, Nat.6)
} by {
    lt_2_9
    lt_3_9
    lt_5_9
    lt_6_9
    ne_2_0
    ne_3_0
    ne_5_0
    ne_6_0
    ne_2_3
    ne_2_5
    ne_2_6
    ne_3_5
    ne_3_6
    ne_5_6
    good_four_intro(Nat.2, Nat.3, Nat.5, Nat.6)
    good_four(Nat.2, Nat.3, Nat.5, Nat.6)
}

/// The good vertices 2, 3, 5, 7 of the nine-element set are a good four.
theorem good_four_2357 {
    good_four(Nat.2, Nat.3, Nat.5, Nat.7)
} by {
    lt_2_9
    lt_3_9
    lt_5_9
    lt_7_9
    ne_2_0
    ne_3_0
    ne_5_0
    ne_7_0
    ne_2_3
    ne_2_5
    ne_2_7
    ne_3_5
    ne_3_7
    ne_5_7
    good_four_intro(Nat.2, Nat.3, Nat.5, Nat.7)
    good_four(Nat.2, Nat.3, Nat.5, Nat.7)
}

/// The good vertices 2, 3, 5, 8 of the nine-element set are a good four.
theorem good_four_2358 {
    good_four(Nat.2, Nat.3, Nat.5, Nat.8)
} by {
    lt_2_9
    lt_3_9
    lt_5_9
    lt_8_9
    ne_2_0
    ne_3_0
    ne_5_0
    ne_8_0
    ne_2_3
    ne_2_5
    ne_2_8
    ne_3_5
    ne_3_8
    ne_5_8
    good_four_intro(Nat.2, Nat.3, Nat.5, Nat.8)
    good_four(Nat.2, Nat.3, Nat.5, Nat.8)
}

/// The good vertices 2, 3, 6, 7 of the nine-element set are a good four.
theorem good_four_2367 {
    good_four(Nat.2, Nat.3, Nat.6, Nat.7)
} by {
    lt_2_9
    lt_3_9
    lt_6_9
    lt_7_9
    ne_2_0
    ne_3_0
    ne_6_0
    ne_7_0
    ne_2_3
    ne_2_6
    ne_2_7
    ne_3_6
    ne_3_7
    ne_6_7
    good_four_intro(Nat.2, Nat.3, Nat.6, Nat.7)
    good_four(Nat.2, Nat.3, Nat.6, Nat.7)
}

/// The good vertices 2, 3, 6, 8 of the nine-element set are a good four.
theorem good_four_2368 {
    good_four(Nat.2, Nat.3, Nat.6, Nat.8)
} by {
    lt_2_9
    lt_3_9
    lt_6_9
    lt_8_9
    ne_2_0
    ne_3_0
    ne_6_0
    ne_8_0
    ne_2_3
    ne_2_6
    ne_2_8
    ne_3_6
    ne_3_8
    ne_6_8
    good_four_intro(Nat.2, Nat.3, Nat.6, Nat.8)
    good_four(Nat.2, Nat.3, Nat.6, Nat.8)
}

/// The good vertices 2, 3, 7, 8 of the nine-element set are a good four.
theorem good_four_2378 {
    good_four(Nat.2, Nat.3, Nat.7, Nat.8)
} by {
    lt_2_9
    lt_3_9
    lt_7_9
    lt_8_9
    ne_2_0
    ne_3_0
    ne_7_0
    ne_8_0
    ne_2_3
    ne_2_7
    ne_2_8
    ne_3_7
    ne_3_8
    ne_7_8
    good_four_intro(Nat.2, Nat.3, Nat.7, Nat.8)
    good_four(Nat.2, Nat.3, Nat.7, Nat.8)
}

/// The good vertices 2, 4, 5, 6 of the nine-element set are a good four.
theorem good_four_2456 {
    good_four(Nat.2, Nat.4, Nat.5, Nat.6)
} by {
    lt_2_9
    lt_4_9
    lt_5_9
    lt_6_9
    ne_2_0
    ne_4_0
    ne_5_0
    ne_6_0
    ne_2_4
    ne_2_5
    ne_2_6
    ne_4_5
    ne_4_6
    ne_5_6
    good_four_intro(Nat.2, Nat.4, Nat.5, Nat.6)
    good_four(Nat.2, Nat.4, Nat.5, Nat.6)
}

/// The good vertices 2, 4, 5, 7 of the nine-element set are a good four.
theorem good_four_2457 {
    good_four(Nat.2, Nat.4, Nat.5, Nat.7)
} by {
    lt_2_9
    lt_4_9
    lt_5_9
    lt_7_9
    ne_2_0
    ne_4_0
    ne_5_0
    ne_7_0
    ne_2_4
    ne_2_5
    ne_2_7
    ne_4_5
    ne_4_7
    ne_5_7
    good_four_intro(Nat.2, Nat.4, Nat.5, Nat.7)
    good_four(Nat.2, Nat.4, Nat.5, Nat.7)
}

/// The good vertices 2, 4, 5, 8 of the nine-element set are a good four.
theorem good_four_2458 {
    good_four(Nat.2, Nat.4, Nat.5, Nat.8)
} by {
    lt_2_9
    lt_4_9
    lt_5_9
    lt_8_9
    ne_2_0
    ne_4_0
    ne_5_0
    ne_8_0
    ne_2_4
    ne_2_5
    ne_2_8
    ne_4_5
    ne_4_8
    ne_5_8
    good_four_intro(Nat.2, Nat.4, Nat.5, Nat.8)
    good_four(Nat.2, Nat.4, Nat.5, Nat.8)
}

/// The good vertices 2, 4, 6, 7 of the nine-element set are a good four.
theorem good_four_2467 {
    good_four(Nat.2, Nat.4, Nat.6, Nat.7)
} by {
    lt_2_9
    lt_4_9
    lt_6_9
    lt_7_9
    ne_2_0
    ne_4_0
    ne_6_0
    ne_7_0
    ne_2_4
    ne_2_6
    ne_2_7
    ne_4_6
    ne_4_7
    ne_6_7
    good_four_intro(Nat.2, Nat.4, Nat.6, Nat.7)
    good_four(Nat.2, Nat.4, Nat.6, Nat.7)
}

/// The good vertices 2, 4, 6, 8 of the nine-element set are a good four.
theorem good_four_2468 {
    good_four(Nat.2, Nat.4, Nat.6, Nat.8)
} by {
    lt_2_9
    lt_4_9
    lt_6_9
    lt_8_9
    ne_2_0
    ne_4_0
    ne_6_0
    ne_8_0
    ne_2_4
    ne_2_6
    ne_2_8
    ne_4_6
    ne_4_8
    ne_6_8
    good_four_intro(Nat.2, Nat.4, Nat.6, Nat.8)
    good_four(Nat.2, Nat.4, Nat.6, Nat.8)
}

/// The good vertices 2, 4, 7, 8 of the nine-element set are a good four.
theorem good_four_2478 {
    good_four(Nat.2, Nat.4, Nat.7, Nat.8)
} by {
    lt_2_9
    lt_4_9
    lt_7_9
    lt_8_9
    ne_2_0
    ne_4_0
    ne_7_0
    ne_8_0
    ne_2_4
    ne_2_7
    ne_2_8
    ne_4_7
    ne_4_8
    ne_7_8
    good_four_intro(Nat.2, Nat.4, Nat.7, Nat.8)
    good_four(Nat.2, Nat.4, Nat.7, Nat.8)
}

/// The good vertices 2, 5, 6, 7 of the nine-element set are a good four.
theorem good_four_2567 {
    good_four(Nat.2, Nat.5, Nat.6, Nat.7)
} by {
    lt_2_9
    lt_5_9
    lt_6_9
    lt_7_9
    ne_2_0
    ne_5_0
    ne_6_0
    ne_7_0
    ne_2_5
    ne_2_6
    ne_2_7
    ne_5_6
    ne_5_7
    ne_6_7
    good_four_intro(Nat.2, Nat.5, Nat.6, Nat.7)
    good_four(Nat.2, Nat.5, Nat.6, Nat.7)
}

/// The good vertices 2, 5, 6, 8 of the nine-element set are a good four.
theorem good_four_2568 {
    good_four(Nat.2, Nat.5, Nat.6, Nat.8)
} by {
    lt_2_9
    lt_5_9
    lt_6_9
    lt_8_9
    ne_2_0
    ne_5_0
    ne_6_0
    ne_8_0
    ne_2_5
    ne_2_6
    ne_2_8
    ne_5_6
    ne_5_8
    ne_6_8
    good_four_intro(Nat.2, Nat.5, Nat.6, Nat.8)
    good_four(Nat.2, Nat.5, Nat.6, Nat.8)
}

/// The good vertices 2, 5, 7, 8 of the nine-element set are a good four.
theorem good_four_2578 {
    good_four(Nat.2, Nat.5, Nat.7, Nat.8)
} by {
    lt_2_9
    lt_5_9
    lt_7_9
    lt_8_9
    ne_2_0
    ne_5_0
    ne_7_0
    ne_8_0
    ne_2_5
    ne_2_7
    ne_2_8
    ne_5_7
    ne_5_8
    ne_7_8
    good_four_intro(Nat.2, Nat.5, Nat.7, Nat.8)
    good_four(Nat.2, Nat.5, Nat.7, Nat.8)
}

/// The good vertices 2, 6, 7, 8 of the nine-element set are a good four.
theorem good_four_2678 {
    good_four(Nat.2, Nat.6, Nat.7, Nat.8)
} by {
    lt_2_9
    lt_6_9
    lt_7_9
    lt_8_9
    ne_2_0
    ne_6_0
    ne_7_0
    ne_8_0
    ne_2_6
    ne_2_7
    ne_2_8
    ne_6_7
    ne_6_8
    ne_7_8
    good_four_intro(Nat.2, Nat.6, Nat.7, Nat.8)
    good_four(Nat.2, Nat.6, Nat.7, Nat.8)
}

/// The good vertices 3, 4, 5, 6 of the nine-element set are a good four.
theorem good_four_3456 {
    good_four(Nat.3, Nat.4, Nat.5, Nat.6)
} by {
    lt_3_9
    lt_4_9
    lt_5_9
    lt_6_9
    ne_3_0
    ne_4_0
    ne_5_0
    ne_6_0
    ne_3_4
    ne_3_5
    ne_3_6
    ne_4_5
    ne_4_6
    ne_5_6
    good_four_intro(Nat.3, Nat.4, Nat.5, Nat.6)
    good_four(Nat.3, Nat.4, Nat.5, Nat.6)
}

/// The good vertices 3, 4, 5, 7 of the nine-element set are a good four.
theorem good_four_3457 {
    good_four(Nat.3, Nat.4, Nat.5, Nat.7)
} by {
    lt_3_9
    lt_4_9
    lt_5_9
    lt_7_9
    ne_3_0
    ne_4_0
    ne_5_0
    ne_7_0
    ne_3_4
    ne_3_5
    ne_3_7
    ne_4_5
    ne_4_7
    ne_5_7
    good_four_intro(Nat.3, Nat.4, Nat.5, Nat.7)
    good_four(Nat.3, Nat.4, Nat.5, Nat.7)
}

/// The good vertices 3, 4, 5, 8 of the nine-element set are a good four.
theorem good_four_3458 {
    good_four(Nat.3, Nat.4, Nat.5, Nat.8)
} by {
    lt_3_9
    lt_4_9
    lt_5_9
    lt_8_9
    ne_3_0
    ne_4_0
    ne_5_0
    ne_8_0
    ne_3_4
    ne_3_5
    ne_3_8
    ne_4_5
    ne_4_8
    ne_5_8
    good_four_intro(Nat.3, Nat.4, Nat.5, Nat.8)
    good_four(Nat.3, Nat.4, Nat.5, Nat.8)
}

/// The good vertices 3, 4, 6, 7 of the nine-element set are a good four.
theorem good_four_3467 {
    good_four(Nat.3, Nat.4, Nat.6, Nat.7)
} by {
    lt_3_9
    lt_4_9
    lt_6_9
    lt_7_9
    ne_3_0
    ne_4_0
    ne_6_0
    ne_7_0
    ne_3_4
    ne_3_6
    ne_3_7
    ne_4_6
    ne_4_7
    ne_6_7
    good_four_intro(Nat.3, Nat.4, Nat.6, Nat.7)
    good_four(Nat.3, Nat.4, Nat.6, Nat.7)
}

/// The good vertices 3, 4, 6, 8 of the nine-element set are a good four.
theorem good_four_3468 {
    good_four(Nat.3, Nat.4, Nat.6, Nat.8)
} by {
    lt_3_9
    lt_4_9
    lt_6_9
    lt_8_9
    ne_3_0
    ne_4_0
    ne_6_0
    ne_8_0
    ne_3_4
    ne_3_6
    ne_3_8
    ne_4_6
    ne_4_8
    ne_6_8
    good_four_intro(Nat.3, Nat.4, Nat.6, Nat.8)
    good_four(Nat.3, Nat.4, Nat.6, Nat.8)
}

/// The good vertices 3, 4, 7, 8 of the nine-element set are a good four.
theorem good_four_3478 {
    good_four(Nat.3, Nat.4, Nat.7, Nat.8)
} by {
    lt_3_9
    lt_4_9
    lt_7_9
    lt_8_9
    ne_3_0
    ne_4_0
    ne_7_0
    ne_8_0
    ne_3_4
    ne_3_7
    ne_3_8
    ne_4_7
    ne_4_8
    ne_7_8
    good_four_intro(Nat.3, Nat.4, Nat.7, Nat.8)
    good_four(Nat.3, Nat.4, Nat.7, Nat.8)
}

/// The good vertices 3, 5, 6, 7 of the nine-element set are a good four.
theorem good_four_3567 {
    good_four(Nat.3, Nat.5, Nat.6, Nat.7)
} by {
    lt_3_9
    lt_5_9
    lt_6_9
    lt_7_9
    ne_3_0
    ne_5_0
    ne_6_0
    ne_7_0
    ne_3_5
    ne_3_6
    ne_3_7
    ne_5_6
    ne_5_7
    ne_6_7
    good_four_intro(Nat.3, Nat.5, Nat.6, Nat.7)
    good_four(Nat.3, Nat.5, Nat.6, Nat.7)
}

/// The good vertices 3, 5, 6, 8 of the nine-element set are a good four.
theorem good_four_3568 {
    good_four(Nat.3, Nat.5, Nat.6, Nat.8)
} by {
    lt_3_9
    lt_5_9
    lt_6_9
    lt_8_9
    ne_3_0
    ne_5_0
    ne_6_0
    ne_8_0
    ne_3_5
    ne_3_6
    ne_3_8
    ne_5_6
    ne_5_8
    ne_6_8
    good_four_intro(Nat.3, Nat.5, Nat.6, Nat.8)
    good_four(Nat.3, Nat.5, Nat.6, Nat.8)
}

/// The good vertices 3, 5, 7, 8 of the nine-element set are a good four.
theorem good_four_3578 {
    good_four(Nat.3, Nat.5, Nat.7, Nat.8)
} by {
    lt_3_9
    lt_5_9
    lt_7_9
    lt_8_9
    ne_3_0
    ne_5_0
    ne_7_0
    ne_8_0
    ne_3_5
    ne_3_7
    ne_3_8
    ne_5_7
    ne_5_8
    ne_7_8
    good_four_intro(Nat.3, Nat.5, Nat.7, Nat.8)
    good_four(Nat.3, Nat.5, Nat.7, Nat.8)
}

/// The good vertices 3, 6, 7, 8 of the nine-element set are a good four.
theorem good_four_3678 {
    good_four(Nat.3, Nat.6, Nat.7, Nat.8)
} by {
    lt_3_9
    lt_6_9
    lt_7_9
    lt_8_9
    ne_3_0
    ne_6_0
    ne_7_0
    ne_8_0
    ne_3_6
    ne_3_7
    ne_3_8
    ne_6_7
    ne_6_8
    ne_7_8
    good_four_intro(Nat.3, Nat.6, Nat.7, Nat.8)
    good_four(Nat.3, Nat.6, Nat.7, Nat.8)
}

/// The good vertices 4, 5, 6, 7 of the nine-element set are a good four.
theorem good_four_4567 {
    good_four(Nat.4, Nat.5, Nat.6, Nat.7)
} by {
    lt_4_9
    lt_5_9
    lt_6_9
    lt_7_9
    ne_4_0
    ne_5_0
    ne_6_0
    ne_7_0
    ne_4_5
    ne_4_6
    ne_4_7
    ne_5_6
    ne_5_7
    ne_6_7
    good_four_intro(Nat.4, Nat.5, Nat.6, Nat.7)
    good_four(Nat.4, Nat.5, Nat.6, Nat.7)
}

/// The good vertices 4, 5, 6, 8 of the nine-element set are a good four.
theorem good_four_4568 {
    good_four(Nat.4, Nat.5, Nat.6, Nat.8)
} by {
    lt_4_9
    lt_5_9
    lt_6_9
    lt_8_9
    ne_4_0
    ne_5_0
    ne_6_0
    ne_8_0
    ne_4_5
    ne_4_6
    ne_4_8
    ne_5_6
    ne_5_8
    ne_6_8
    good_four_intro(Nat.4, Nat.5, Nat.6, Nat.8)
    good_four(Nat.4, Nat.5, Nat.6, Nat.8)
}

/// The good vertices 4, 5, 7, 8 of the nine-element set are a good four.
theorem good_four_4578 {
    good_four(Nat.4, Nat.5, Nat.7, Nat.8)
} by {
    lt_4_9
    lt_5_9
    lt_7_9
    lt_8_9
    ne_4_0
    ne_5_0
    ne_7_0
    ne_8_0
    ne_4_5
    ne_4_7
    ne_4_8
    ne_5_7
    ne_5_8
    ne_7_8
    good_four_intro(Nat.4, Nat.5, Nat.7, Nat.8)
    good_four(Nat.4, Nat.5, Nat.7, Nat.8)
}

/// The good vertices 4, 6, 7, 8 of the nine-element set are a good four.
theorem good_four_4678 {
    good_four(Nat.4, Nat.6, Nat.7, Nat.8)
} by {
    lt_4_9
    lt_6_9
    lt_7_9
    lt_8_9
    ne_4_0
    ne_6_0
    ne_7_0
    ne_8_0
    ne_4_6
    ne_4_7
    ne_4_8
    ne_6_7
    ne_6_8
    ne_7_8
    good_four_intro(Nat.4, Nat.6, Nat.7, Nat.8)
    good_four(Nat.4, Nat.6, Nat.7, Nat.8)
}

/// The good vertices 5, 6, 7, 8 of the nine-element set are a good four.
theorem good_four_5678 {
    good_four(Nat.5, Nat.6, Nat.7, Nat.8)
} by {
    lt_5_9
    lt_6_9
    lt_7_9
    lt_8_9
    ne_5_0
    ne_6_0
    ne_7_0
    ne_8_0
    ne_5_6
    ne_5_7
    ne_5_8
    ne_6_7
    ne_6_8
    ne_7_8
    good_four_intro(Nat.5, Nat.6, Nat.7, Nat.8)
    good_four(Nat.5, Nat.6, Nat.7, Nat.8)
}
