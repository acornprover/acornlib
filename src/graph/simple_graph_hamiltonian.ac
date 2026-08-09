from nat import Nat, alt_induction, not_lt_zero, lt_and_lte, lt_not_ref
from data.basic.relation_basic import is_symmetric, is_irreflexive
from data.basic.logic import false_implies
from list import List, unique_list_sum, singleton_unique, singleton_contains_imp_eq
from data.list.list_cons_membership import cons_contains_eq, cons_contains_head,
    cons_contains_of_tail_contains, nil_not_contains
from data.nat.nat_range_set import range_set, range_set_contains_eq, range_set_contains
from finite_set import FiniteSet
from pair import Pair
from graph.simple_graph import SimpleGraph, complete_graph, complete_graph_adj_iff_ne,
    simple_graph_adj_comm
from graph.simple_graph_walks import simple_graph_walk, simple_graph_walk_vertices,
    simple_graph_closed_walk, simple_graph_closed_walk_intro, simple_graph_walk_nil,
    simple_graph_walk_cons_iff, simple_graph_closed_walk_is_walk
from graph.simple_graph_eulerian import walk_vertices_in_set
from graph.simple_graph_connectivity import simple_graph_reachable, simple_graph_reachable_of_adj,
    simple_graph_reachable_refl, simple_graph_reachable_symmetric, simple_graph_reachable_transitive
from graph.simple_graph_cycle import cycle_graph, cycle_graph_adj_suc, cycle_graph_adj_wrap

numerals Nat

/// True when the walk's vertex list contains every vertex of `s`.
///
/// This is the covering half of the Hamiltonian condition: paired with
/// `walk_vertices_in_set` (the walk stays inside `s`) it says the walk's vertex list
/// is exactly the vertex set, and with the uniqueness condition each vertex is hit
/// exactly once.
define walk_visits_set[V](s: FiniteSet[V], start: V, steps: List[V]) -> Bool {
    forall(x: V) {
        s.contains(x) implies simple_graph_walk_vertices(start, steps).contains(x)
    }
}

/// A Hamiltonian cycle of `(g, s)`: a cycle visiting every vertex of `s` exactly once.
///
/// The cycle is stored as a closed walk `(start, steps)`, matching `simple_graph_walk`.
/// A closed walk's target list ends with `start` (the closing step), so the vertices
/// visited are `start` followed by the targets; requiring the target list to have no
/// repeats forces every visit other than the closing return to `start` to hit a fresh
/// vertex. The walk staying inside `s` and visiting all of `s` then makes the visited
/// vertices exactly `s`, each exactly once. The lower bound on the length rules out
/// the degenerate two-edge closed walk of a simple graph, which visits only two
/// vertices.
define is_hamiltonian_cycle[V](g: SimpleGraph[V], s: FiniteSet[V], start: V, steps: List[V]) -> Bool {
    simple_graph_closed_walk(g, start, steps) and
    Nat.3 <= steps.length and
    steps.is_unique and
    walk_vertices_in_set(s, start, steps) and
    walk_visits_set(s, start, steps)
}

/// The defining equation of `is_hamiltonian_cycle`, packaged for explicit citation.
///
/// Proof search unfolds `define`d predicates slowly, and with a symbolic walk the
/// arithmetic conjunct can make it time out, so the equation is stated once here.
theorem is_hamiltonian_cycle_unfold[V](g: SimpleGraph[V], s: FiniteSet[V], start: V, steps: List[V]) {
    is_hamiltonian_cycle(g, s, start, steps) = (
        simple_graph_closed_walk(g, start, steps) and Nat.3 <= steps.length and
        steps.is_unique and walk_vertices_in_set(s, start, steps) and
        walk_visits_set(s, start, steps)
    )
} by {
    is_hamiltonian_cycle(g, s, start, steps) = (
        simple_graph_closed_walk(g, start, steps) and Nat.3 <= steps.length and
        steps.is_unique and walk_vertices_in_set(s, start, steps) and
        walk_visits_set(s, start, steps)
    )
}

/// A finite vertex set is connected when any two of its vertices are reachable.
///
/// The library's `simple_graph_set_connected` ranges over the general `Set` type;
/// this is the same notion specialized to the finite vertex sets that the graph
/// predicates here carry.
define simple_graph_fs_connected[V](g: SimpleGraph[V], s: FiniteSet[V]) -> Bool {
    forall(x: V, y: V) {
        s.contains(x) and s.contains(y) implies simple_graph_reachable(g, x, y)
    }
}

/// True when `(g, s)` has a Hamiltonian cycle.
define is_hamiltonian[V](g: SimpleGraph[V], s: FiniteSet[V]) -> Bool {
    exists(start: V, steps: List[V]) {
        is_hamiltonian_cycle(g, s, start, steps)
    }
}

/// The targets of the counterclockwise Hamiltonian walk of the cycle graph: `n - 1`, ..., `1`, `0`.
///
/// Walking `0 -> (n-1) -> (n-2) -> ... -> 1 -> 0` traverses every edge of `C_n` in
/// reverse order and returns to the start, so this single recursive definition serves
/// the whole family of cycle graphs at once.
define cycle_steps(n: Nat) -> List[Nat] {
    match n {
        Nat.zero {
            List.nil[Nat]
        }
        Nat.suc(m) {
            List.cons(m, cycle_steps(m))
        }
    }
}

/// The list induction principle, packaged as an eliminator.
lemma list_induction_elim[T](p: List[T] -> Bool, items: List[T]) {
    p(List.nil[T]) and forall(head: T, tail: List[T]) { p(tail) implies p(List.cons[T](head, tail)) }
    implies p(items)
} by {
    if p(List.nil[T]) and forall(head: T, tail: List[T]) { p(tail) implies p(List.cons[T](head, tail)) } {
        List.induction(p)
        forall(xs: List[T]) { p(xs) }
        p(items)
    }
}

/// Every target of `cycle_steps(n)` lies below `n`.
theorem cycle_steps_lt(n: Nat, x: Nat) {
    cycle_steps(n).contains(x) implies x < n
} by {
    define p(k: Nat) -> Bool {
        forall(y: Nat) {
            cycle_steps(k).contains(y) implies y < k
        }
    }

    forall(y: Nat) {
        cycle_steps(Nat.0) = List.nil[Nat]
        nil_not_contains(y)
        not List.nil[Nat].contains(y)
        (cycle_steps(Nat.0).contains(y) and y = y) = false
        false_implies(y < Nat.0)
        (false implies y < Nat.0) = true
        ((cycle_steps(Nat.0).contains(y) and y = y) implies y < Nat.0) = true
        (cycle_steps(Nat.0).contains(y) and y = y) implies y < Nat.0
        cycle_steps(Nat.0).contains(y) implies y < Nat.0
    }
    p(Nat.0)

    forall(k: Nat) {
        if p(k) {
            forall(y: Nat) {
                if cycle_steps(k.suc).contains(y) {
                    cycle_steps(k.suc) = List.cons(k, cycle_steps(k))
                    cons_contains_eq(k, cycle_steps(k), y)
                    List.cons(k, cycle_steps(k)).contains(y) = (k = y or cycle_steps(k).contains(y))
                    if y = k {
                        k < k.suc
                        y < k.suc
                    }
                    if y != k {
                        cycle_steps(k).contains(y)
                        p(k) = forall(z: Nat) {
                            cycle_steps(k).contains(z) implies z < k
                        }
                        y < k
                        k < k.suc
                        y < k.suc
                    }
                    y < k.suc
                }
                cycle_steps(k.suc).contains(y) implies y < k.suc
            }
            p(k.suc)
        }
        p(k) implies p(k.suc)
    }
    forall(k: Nat) { p(k) implies p(k.suc) }
    p(Nat.0) and forall(k: Nat) { p(k) implies p(k.suc) }
    alt_induction(p)
    forall(k: Nat) { p(k) }
    p(n)
    p(n) = forall(y: Nat) {
        cycle_steps(n).contains(y) implies y < n
    }
    if cycle_steps(n).contains(x) {
        x < n
    }
}

/// The length of `cycle_steps(n)` is `n`.
theorem cycle_steps_length(n: Nat) {
    cycle_steps(n).length = n
} by {
    define p(k: Nat) -> Bool {
        cycle_steps(k).length = k
    }

    cycle_steps(Nat.0) = List.nil[Nat]
    List.nil[Nat].length = Nat.0
    cycle_steps(Nat.0).length = Nat.0
    p(Nat.0)

    forall(k: Nat) {
        if p(k) {
            cycle_steps(k.suc) = List.cons(k, cycle_steps(k))
            List.cons(k, cycle_steps(k)).length = cycle_steps(k).length + Nat.1
            p(k) = (cycle_steps(k).length = k)
            cycle_steps(k).length = k
            cycle_steps(k.suc).length = k + Nat.1
            k + Nat.1 = k.suc
            cycle_steps(k.suc).length = k.suc
            p(k.suc)
        }
        p(k) implies p(k.suc)
    }
    forall(k: Nat) { p(k) implies p(k.suc) }
    p(Nat.0) and forall(k: Nat) { p(k) implies p(k.suc) }
    alt_induction(p)
    forall(k: Nat) { p(k) }
    p(n)
    p(n) = (cycle_steps(n).length = n)
    cycle_steps(n).length = n
}

/// `cycle_steps(n)` has no repeated targets.
theorem cycle_steps_unique(n: Nat) {
    cycle_steps(n).is_unique
} by {
    define p(k: Nat) -> Bool {
        cycle_steps(k).is_unique
    }

    cycle_steps(Nat.0) = List.nil[Nat]
    List.nil[Nat].unique = List.nil[Nat]
    List.nil[Nat].is_unique = (List.nil[Nat].unique = List.nil[Nat])
    List.nil[Nat].is_unique
    cycle_steps(Nat.0).is_unique
    p(Nat.0)

    forall(k: Nat) {
        if p(k) {
            cycle_steps(k.suc) = List.cons(k, cycle_steps(k))
            forall(x: Nat) {
                if List.singleton(k).contains(x) and cycle_steps(k).contains(x) {
                    singleton_contains_imp_eq(k, x)
                    x = k
                    cycle_steps_lt(k, x)
                    cycle_steps(k).contains(x) implies x < k
                    x < k
                    k < k
                    false
                }
                not (List.singleton(k).contains(x) and cycle_steps(k).contains(x))
            }
            unique_list_sum(List.singleton(k), cycle_steps(k))
            singleton_unique(k)
            List.singleton(k).is_unique
            p(k) = (cycle_steps(k).is_unique)
            cycle_steps(k).is_unique
            List.singleton(k).is_unique and cycle_steps(k).is_unique
            List.singleton(k).is_unique and cycle_steps(k).is_unique and
                (forall(x: Nat) { not (List.singleton(k).contains(x) and cycle_steps(k).contains(x)) })
            (List.singleton(k) + cycle_steps(k)).is_unique
            List.singleton(k) + cycle_steps(k) = List.cons(k, cycle_steps(k))
            List.cons(k, cycle_steps(k)).is_unique
            cycle_steps(k.suc).is_unique
            p(k.suc)
        }
        p(k) implies p(k.suc)
    }
    forall(k: Nat) { p(k) implies p(k.suc) }
    p(Nat.0) and forall(k: Nat) { p(k) implies p(k.suc) }
    alt_induction(p)
    forall(k: Nat) { p(k) }
    p(n)
    p(n) = (cycle_steps(n).is_unique)
    cycle_steps(n).is_unique
}

/// `cycle_steps(n)` contains every positive vertex below `n`.
theorem cycle_steps_contains_of_lt(n: Nat, x: Nat) {
    Nat.1 <= x and x < n implies cycle_steps(n).contains(x)
} by {
    define p(k: Nat) -> Bool {
        forall(y: Nat) {
            Nat.1 <= y and y < k implies cycle_steps(k).contains(y)
        }
    }

    forall(y: Nat) {
        not_lt_zero(y)
        not (y < Nat.0)
        (Nat.1 <= y and y < Nat.0) = false
        false_implies(cycle_steps(Nat.0).contains(y))
        (false implies cycle_steps(Nat.0).contains(y)) = true
        ((Nat.1 <= y and y < Nat.0) implies cycle_steps(Nat.0).contains(y)) = true
        (Nat.1 <= y and y < Nat.0) implies cycle_steps(Nat.0).contains(y)
    }
    p(Nat.0)

    forall(k: Nat) {
        if p(k) {
            forall(y: Nat) {
                if Nat.1 <= y and y < k.suc {
                    cycle_steps(k.suc) = List.cons(k, cycle_steps(k))
                    if y = k {
                        cons_contains_head(k, cycle_steps(k))
                        List.cons(k, cycle_steps(k)).contains(k)
                        List.cons(k, cycle_steps(k)).contains(y)
                        cycle_steps(k.suc).contains(y)
                    }
                    if y != k {
                        y < k
                        p(k) = forall(z: Nat) {
                            Nat.1 <= z and z < k implies cycle_steps(k).contains(z)
                        }
                        cycle_steps(k).contains(y)
                        cons_contains_of_tail_contains(k, cycle_steps(k), y)
                        List.cons(k, cycle_steps(k)).contains(y)
                        cycle_steps(k.suc).contains(y)
                    }
                    cycle_steps(k.suc).contains(y)
                }
                Nat.1 <= y and y < k.suc implies cycle_steps(k.suc).contains(y)
            }
            p(k.suc)
        }
        p(k) implies p(k.suc)
    }
    forall(k: Nat) { p(k) implies p(k.suc) }
    p(Nat.0) and forall(k: Nat) { p(k) implies p(k.suc) }
    alt_induction(p)
    forall(k: Nat) { p(k) }
    p(n)
    p(n) = forall(y: Nat) {
        Nat.1 <= y and y < n implies cycle_steps(n).contains(y)
    }
    if Nat.1 <= x and x < n {
        cycle_steps(n).contains(x)
    }
}

/// The counterclockwise walk of the cycle graph stays inside `{0, ..., n - 1}`.
theorem cycle_steps_walk_vertices_in_range(n: Nat) {
    Nat.1 <= n implies walk_vertices_in_set(range_set(n), Nat.0, cycle_steps(n))
} by {
    if Nat.1 <= n {
        walk_vertices_in_set(range_set(n), Nat.0, cycle_steps(n)) = forall(x: Nat) {
            simple_graph_walk_vertices(Nat.0, cycle_steps(n)).contains(x) implies range_set(n).contains(x)
        }
        forall(x: Nat) {
            if simple_graph_walk_vertices(Nat.0, cycle_steps(n)).contains(x) {
                simple_graph_walk_vertices(Nat.0, cycle_steps(n)) = List.cons(Nat.0, cycle_steps(n))
                cons_contains_eq(Nat.0, cycle_steps(n), x)
                List.cons(Nat.0, cycle_steps(n)).contains(x) = (Nat.0 = x or cycle_steps(n).contains(x))
                if x = Nat.0 {
                    Nat.0 < Nat.1
                    lt_and_lte(Nat.0, Nat.1, n)
                    Nat.0 < n
                    x < n
                    range_set_contains(n, x)
                    range_set(n).contains(x)
                }
                if x != Nat.0 {
                    cycle_steps(n).contains(x)
                    cycle_steps_lt(n, x)
                    x < n
                    range_set_contains(n, x)
                    range_set(n).contains(x)
                }
                range_set(n).contains(x)
            }
            simple_graph_walk_vertices(Nat.0, cycle_steps(n)).contains(x) implies range_set(n).contains(x)
        }
        walk_vertices_in_set(range_set(n), Nat.0, cycle_steps(n))
    }
}

/// The counterclockwise walk of the cycle graph visits every vertex of `{0, ..., n - 1}`.
theorem cycle_steps_walk_visits_range(n: Nat) {
    walk_visits_set(range_set(n), Nat.0, cycle_steps(n))
} by {
    walk_visits_set(range_set(n), Nat.0, cycle_steps(n)) = forall(x: Nat) {
        range_set(n).contains(x) implies simple_graph_walk_vertices(Nat.0, cycle_steps(n)).contains(x)
    }
    forall(x: Nat) {
        if range_set(n).contains(x) {
            range_set_contains_eq(n, x)
            range_set(n).contains(x) = (x < n)
            x < n
            simple_graph_walk_vertices(Nat.0, cycle_steps(n)) = List.cons(Nat.0, cycle_steps(n))
            if x = Nat.0 {
                cons_contains_head(Nat.0, cycle_steps(n))
                List.cons(Nat.0, cycle_steps(n)).contains(Nat.0)
                simple_graph_walk_vertices(Nat.0, cycle_steps(n)).contains(x)
            }
            if x != Nat.0 {
                Nat.1 <= x
                cycle_steps_contains_of_lt(n, x)
                cycle_steps(n).contains(x)
                cons_contains_of_tail_contains(Nat.0, cycle_steps(n), x)
                List.cons(Nat.0, cycle_steps(n)).contains(x)
                simple_graph_walk_vertices(Nat.0, cycle_steps(n)).contains(x)
            }
            simple_graph_walk_vertices(Nat.0, cycle_steps(n)).contains(x)
        }
        range_set(n).contains(x) implies simple_graph_walk_vertices(Nat.0, cycle_steps(n)).contains(x)
    }
    walk_visits_set(range_set(n), Nat.0, cycle_steps(n))
}

/// Walking down from `m` along consecutive vertices of the cycle graph reaches `0`.
///
/// The walk `m -> (m-1) -> ... -> 1 -> 0` uses only the successor edges of `C_n`,
/// which exist below `n`; the wrap-around edge is added separately by the caller.
theorem cycle_graph_walk_desc(n: Nat, m: Nat) {
    m < n implies simple_graph_walk(cycle_graph(n), m, Nat.0, cycle_steps(m))
} by {
    define p(k: Nat) -> Bool {
        forall(nn: Nat) {
            k < nn implies simple_graph_walk(cycle_graph(nn), k, Nat.0, cycle_steps(k))
        }
    }

    forall(nn: Nat) {
        if Nat.0 < nn {
            simple_graph_walk_nil(cycle_graph(nn), Nat.0, Nat.0)
            simple_graph_walk(cycle_graph(nn), Nat.0, Nat.0, List.nil[Nat])
            cycle_steps(Nat.0) = List.nil[Nat]
            simple_graph_walk(cycle_graph(nn), Nat.0, Nat.0, cycle_steps(Nat.0))
        }
        Nat.0 < nn implies simple_graph_walk(cycle_graph(nn), Nat.0, Nat.0, cycle_steps(Nat.0))
    }
    p(Nat.0)

    forall(k: Nat) {
        if p(k) {
            forall(nn: Nat) {
                if k.suc < nn {
                    cycle_steps(k.suc) = List.cons(k, cycle_steps(k))
                    simple_graph_walk_cons_iff(cycle_graph(nn), k.suc, Nat.0, k, cycle_steps(k))
                    simple_graph_walk(cycle_graph(nn), k.suc, Nat.0, List.cons(k, cycle_steps(k))) =
                        (cycle_graph(nn).adj(k.suc, k) and
                            simple_graph_walk(cycle_graph(nn), k, Nat.0, cycle_steps(k)))
                    cycle_graph_adj_suc(nn, k)
                    k.suc < nn
                    cycle_graph(nn).adj(k, k.suc)
                    simple_graph_adj_comm(cycle_graph(nn), k, k.suc)
                    cycle_graph(nn).adj(k.suc, k) = cycle_graph(nn).adj(k, k.suc)
                    cycle_graph(nn).adj(k.suc, k)
                    p(k) = forall(m0: Nat) {
                        k < m0 implies simple_graph_walk(cycle_graph(m0), k, Nat.0, cycle_steps(k))
                    }
                    k < nn
                    simple_graph_walk(cycle_graph(nn), k, Nat.0, cycle_steps(k))
                    cycle_graph(nn).adj(k.suc, k) and
                        simple_graph_walk(cycle_graph(nn), k, Nat.0, cycle_steps(k))
                    simple_graph_walk(cycle_graph(nn), k.suc, Nat.0, List.cons(k, cycle_steps(k)))
                    simple_graph_walk(cycle_graph(nn), k.suc, Nat.0, cycle_steps(k.suc))
                }
                k.suc < nn implies simple_graph_walk(cycle_graph(nn), k.suc, Nat.0, cycle_steps(k.suc))
            }
            p(k.suc)
        }
        p(k) implies p(k.suc)
    }
    forall(k: Nat) { p(k) implies p(k.suc) }
    p(Nat.0) and forall(k: Nat) { p(k) implies p(k.suc) }
    alt_induction(p)
    forall(k: Nat) { p(k) }
    p(m)
    p(m) = forall(nn: Nat) {
        m < nn implies simple_graph_walk(cycle_graph(nn), m, Nat.0, cycle_steps(m))
    }
    if m < n {
        simple_graph_walk(cycle_graph(n), m, Nat.0, cycle_steps(m))
    }
}

/// The counterclockwise walk of the cycle graph is closed: the wrap-around edge returns to `0`.
theorem cycle_graph_closed_walk_suc(m: Nat) {
    Nat.2 <= m implies simple_graph_closed_walk(cycle_graph(m.suc), Nat.0, cycle_steps(m.suc))
} by {
    if Nat.2 <= m {
        cycle_steps(m.suc) = List.cons(m, cycle_steps(m))
        simple_graph_walk_cons_iff(cycle_graph(m.suc), Nat.0, Nat.0, m, cycle_steps(m))
        simple_graph_walk(cycle_graph(m.suc), Nat.0, Nat.0, List.cons(m, cycle_steps(m))) =
            (cycle_graph(m.suc).adj(Nat.0, m) and
                simple_graph_walk(cycle_graph(m.suc), m, Nat.0, cycle_steps(m)))
        cycle_graph_adj_wrap(m)
        cycle_graph(m.suc).adj(m, Nat.0)
        simple_graph_adj_comm(cycle_graph(m.suc), Nat.0, m)
        cycle_graph(m.suc).adj(Nat.0, m) = cycle_graph(m.suc).adj(m, Nat.0)
        cycle_graph(m.suc).adj(Nat.0, m)
        cycle_graph_walk_desc(m.suc, m)
        m < m.suc
        simple_graph_walk(cycle_graph(m.suc), m, Nat.0, cycle_steps(m))
        cycle_graph(m.suc).adj(Nat.0, m) and
            simple_graph_walk(cycle_graph(m.suc), m, Nat.0, cycle_steps(m))
        simple_graph_walk(cycle_graph(m.suc), Nat.0, Nat.0, List.cons(m, cycle_steps(m)))
        simple_graph_walk(cycle_graph(m.suc), Nat.0, Nat.0, cycle_steps(m.suc))
        cycle_steps_length(m.suc)
        cycle_steps(m.suc).length = m.suc
        m.suc > Nat.0
        cycle_steps(m.suc).length > Nat.0
        simple_graph_closed_walk_intro(cycle_graph(m.suc), Nat.0, cycle_steps(m.suc))
        simple_graph_closed_walk(cycle_graph(m.suc), Nat.0, cycle_steps(m.suc))
    }
}

/// Walking down from `m` along the complete graph reaches `0`.
///
/// In the complete graph every pair of distinct vertices is adjacent, so the same
/// descending walk that works on the cycle graph is a walk here as well.
theorem complete_graph_walk_desc(n: Nat, m: Nat) {
    m < n implies simple_graph_walk(complete_graph[Nat], m, Nat.0, cycle_steps(m))
} by {
    define p(k: Nat) -> Bool {
        forall(nn: Nat) {
            k < nn implies simple_graph_walk(complete_graph[Nat], k, Nat.0, cycle_steps(k))
        }
    }

    forall(nn: Nat) {
        if Nat.0 < nn {
            simple_graph_walk_nil(complete_graph[Nat], Nat.0, Nat.0)
            simple_graph_walk(complete_graph[Nat], Nat.0, Nat.0, List.nil[Nat])
            cycle_steps(Nat.0) = List.nil[Nat]
            simple_graph_walk(complete_graph[Nat], Nat.0, Nat.0, cycle_steps(Nat.0))
        }
        Nat.0 < nn implies simple_graph_walk(complete_graph[Nat], Nat.0, Nat.0, cycle_steps(Nat.0))
    }
    p(Nat.0)

    forall(k: Nat) {
        if p(k) {
            forall(nn: Nat) {
                if k.suc < nn {
                    cycle_steps(k.suc) = List.cons(k, cycle_steps(k))
                    simple_graph_walk_cons_iff(complete_graph[Nat], k.suc, Nat.0, k, cycle_steps(k))
                    simple_graph_walk(complete_graph[Nat], k.suc, Nat.0, List.cons(k, cycle_steps(k))) =
                        (complete_graph[Nat].adj(k.suc, k) and
                            simple_graph_walk(complete_graph[Nat], k, Nat.0, cycle_steps(k)))
                    complete_graph_adj_iff_ne[Nat](k.suc, k)
                    complete_graph[Nat].adj(k.suc, k) = (k.suc != k)
                    k.suc != k
                    complete_graph[Nat].adj(k.suc, k)
                    p(k) = forall(m0: Nat) {
                        k < m0 implies simple_graph_walk(complete_graph[Nat], k, Nat.0, cycle_steps(k))
                    }
                    k < nn
                    simple_graph_walk(complete_graph[Nat], k, Nat.0, cycle_steps(k))
                    complete_graph[Nat].adj(k.suc, k) and
                        simple_graph_walk(complete_graph[Nat], k, Nat.0, cycle_steps(k))
                    simple_graph_walk(complete_graph[Nat], k.suc, Nat.0, List.cons(k, cycle_steps(k)))
                    simple_graph_walk(complete_graph[Nat], k.suc, Nat.0, cycle_steps(k.suc))
                }
                k.suc < nn implies simple_graph_walk(complete_graph[Nat], k.suc, Nat.0, cycle_steps(k.suc))
            }
            p(k.suc)
        }
        p(k) implies p(k.suc)
    }
    forall(k: Nat) { p(k) implies p(k.suc) }
    p(Nat.0) and forall(k: Nat) { p(k) implies p(k.suc) }
    alt_induction(p)
    forall(k: Nat) { p(k) }
    p(m)
    p(m) = forall(nn: Nat) {
        m < nn implies simple_graph_walk(complete_graph[Nat], m, Nat.0, cycle_steps(m))
    }
    if m < n {
        simple_graph_walk(complete_graph[Nat], m, Nat.0, cycle_steps(m))
    }
}

/// The counterclockwise walk of the complete graph is closed.
theorem complete_graph_closed_walk_suc(m: Nat) {
    Nat.2 <= m implies simple_graph_closed_walk(complete_graph[Nat], Nat.0, cycle_steps(m.suc))
} by {
    if Nat.2 <= m {
        cycle_steps(m.suc) = List.cons(m, cycle_steps(m))
        simple_graph_walk_cons_iff(complete_graph[Nat], Nat.0, Nat.0, m, cycle_steps(m))
        simple_graph_walk(complete_graph[Nat], Nat.0, Nat.0, List.cons(m, cycle_steps(m))) =
            (complete_graph[Nat].adj(Nat.0, m) and
                simple_graph_walk(complete_graph[Nat], m, Nat.0, cycle_steps(m)))
        Nat.0 < Nat.2
        lt_and_lte(Nat.0, Nat.2, m)
        Nat.0 < m
        Nat.0 != m
        complete_graph_adj_iff_ne[Nat](Nat.0, m)
        complete_graph[Nat].adj(Nat.0, m) = (Nat.0 != m)
        complete_graph[Nat].adj(Nat.0, m)
        complete_graph_walk_desc(m.suc, m)
        m < m.suc
        simple_graph_walk(complete_graph[Nat], m, Nat.0, cycle_steps(m))
        complete_graph[Nat].adj(Nat.0, m) and
            simple_graph_walk(complete_graph[Nat], m, Nat.0, cycle_steps(m))
        simple_graph_walk(complete_graph[Nat], Nat.0, Nat.0, List.cons(m, cycle_steps(m)))
        simple_graph_walk(complete_graph[Nat], Nat.0, Nat.0, cycle_steps(m.suc))
        cycle_steps_length(m.suc)
        cycle_steps(m.suc).length = m.suc
        m.suc > Nat.0
        cycle_steps(m.suc).length > Nat.0
        simple_graph_closed_walk_intro(complete_graph[Nat], Nat.0, cycle_steps(m.suc))
        simple_graph_closed_walk(complete_graph[Nat], Nat.0, cycle_steps(m.suc))
    }
}

/// The cycle graph on at least three vertices is Hamiltonian.
///
/// The counterclockwise walk `0 -> (n-1) -> ... -> 1 -> 0` is a closed walk whose
/// target list has no repeats and covers exactly the vertex set, so it is a
/// Hamiltonian cycle.
theorem cycle_graph_suc_is_hamiltonian(m: Nat) {
    Nat.2 <= m implies is_hamiltonian(cycle_graph(m.suc), range_set(m.suc))
} by {
    if Nat.2 <= m {
        is_hamiltonian_cycle_unfold(cycle_graph(m.suc), range_set(m.suc), Nat.0, cycle_steps(m.suc))
        is_hamiltonian_cycle(cycle_graph(m.suc), range_set(m.suc), Nat.0, cycle_steps(m.suc)) = (
            simple_graph_closed_walk(cycle_graph(m.suc), Nat.0, cycle_steps(m.suc)) and
            Nat.3 <= cycle_steps(m.suc).length and
            cycle_steps(m.suc).is_unique and
            walk_vertices_in_set(range_set(m.suc), Nat.0, cycle_steps(m.suc)) and
            walk_visits_set(range_set(m.suc), Nat.0, cycle_steps(m.suc))
        )
        cycle_graph_closed_walk_suc(m)
        simple_graph_closed_walk(cycle_graph(m.suc), Nat.0, cycle_steps(m.suc))
        cycle_steps_length(m.suc)
        cycle_steps(m.suc).length = m.suc
        Nat.3 <= m.suc
        Nat.3 <= cycle_steps(m.suc).length
        cycle_steps_unique(m.suc)
        cycle_steps(m.suc).is_unique
        Nat.1 <= m.suc
        cycle_steps_walk_vertices_in_range(m.suc)
        walk_vertices_in_set(range_set(m.suc), Nat.0, cycle_steps(m.suc))
        cycle_steps_walk_visits_range(m.suc)
        walk_visits_set(range_set(m.suc), Nat.0, cycle_steps(m.suc))
        simple_graph_closed_walk(cycle_graph(m.suc), Nat.0, cycle_steps(m.suc)) and
            Nat.3 <= cycle_steps(m.suc).length and
            cycle_steps(m.suc).is_unique and
            walk_vertices_in_set(range_set(m.suc), Nat.0, cycle_steps(m.suc)) and
            walk_visits_set(range_set(m.suc), Nat.0, cycle_steps(m.suc))
        is_hamiltonian_cycle(cycle_graph(m.suc), range_set(m.suc), Nat.0, cycle_steps(m.suc))
        is_hamiltonian(cycle_graph(m.suc), range_set(m.suc)) = exists(start: Nat, steps: List[Nat]) {
            is_hamiltonian_cycle(cycle_graph(m.suc), range_set(m.suc), start, steps)
        }
        is_hamiltonian(cycle_graph(m.suc), range_set(m.suc))
    }
}

/// The complete graph on at least three vertices is Hamiltonian.
///
/// The same counterclockwise walk is a Hamiltonian cycle of `K_n`: every consecutive
/// pair is a pair of distinct vertices, hence an edge of the complete graph.
theorem complete_graph_suc_is_hamiltonian(m: Nat) {
    Nat.2 <= m implies is_hamiltonian(complete_graph[Nat], range_set(m.suc))
} by {
    if Nat.2 <= m {
        is_hamiltonian_cycle_unfold(complete_graph[Nat], range_set(m.suc), Nat.0, cycle_steps(m.suc))
        is_hamiltonian_cycle(complete_graph[Nat], range_set(m.suc), Nat.0, cycle_steps(m.suc)) = (
            simple_graph_closed_walk(complete_graph[Nat], Nat.0, cycle_steps(m.suc)) and
            Nat.3 <= cycle_steps(m.suc).length and
            cycle_steps(m.suc).is_unique and
            walk_vertices_in_set(range_set(m.suc), Nat.0, cycle_steps(m.suc)) and
            walk_visits_set(range_set(m.suc), Nat.0, cycle_steps(m.suc))
        )
        complete_graph_closed_walk_suc(m)
        simple_graph_closed_walk(complete_graph[Nat], Nat.0, cycle_steps(m.suc))
        cycle_steps_length(m.suc)
        cycle_steps(m.suc).length = m.suc
        Nat.3 <= m.suc
        Nat.3 <= cycle_steps(m.suc).length
        cycle_steps_unique(m.suc)
        cycle_steps(m.suc).is_unique
        Nat.1 <= m.suc
        cycle_steps_walk_vertices_in_range(m.suc)
        walk_vertices_in_set(range_set(m.suc), Nat.0, cycle_steps(m.suc))
        cycle_steps_walk_visits_range(m.suc)
        walk_visits_set(range_set(m.suc), Nat.0, cycle_steps(m.suc))
        simple_graph_closed_walk(complete_graph[Nat], Nat.0, cycle_steps(m.suc)) and
            Nat.3 <= cycle_steps(m.suc).length and
            cycle_steps(m.suc).is_unique and
            walk_vertices_in_set(range_set(m.suc), Nat.0, cycle_steps(m.suc)) and
            walk_visits_set(range_set(m.suc), Nat.0, cycle_steps(m.suc))
        is_hamiltonian_cycle(complete_graph[Nat], range_set(m.suc), Nat.0, cycle_steps(m.suc))
        is_hamiltonian(complete_graph[Nat], range_set(m.suc)) = exists(start: Nat, steps: List[Nat]) {
            is_hamiltonian_cycle(complete_graph[Nat], range_set(m.suc), start, steps)
        }
        is_hamiltonian(complete_graph[Nat], range_set(m.suc))
    }
}

/// The three-cycle is Hamiltonian.
theorem cycle_graph_3_is_hamiltonian {
    is_hamiltonian(cycle_graph(Nat.3), range_set(Nat.3))
} by {
    cycle_graph_suc_is_hamiltonian(Nat.2)
    Nat.2 <= Nat.2
    is_hamiltonian(cycle_graph(Nat.2.suc), range_set(Nat.2.suc))
    cycle_graph(Nat.2.suc) = cycle_graph(Nat.3)
    range_set(Nat.2.suc) = range_set(Nat.3)
    is_hamiltonian(cycle_graph(Nat.3), range_set(Nat.3))
}

/// The four-cycle is Hamiltonian.
theorem cycle_graph_4_is_hamiltonian {
    is_hamiltonian(cycle_graph(Nat.4), range_set(Nat.4))
} by {
    cycle_graph_suc_is_hamiltonian(Nat.3)
    Nat.2.suc = Nat.3
    Nat.2 < Nat.2.suc
    Nat.2 < Nat.3
    Nat.2 <= Nat.3
    is_hamiltonian(cycle_graph(Nat.3.suc), range_set(Nat.3.suc))
    cycle_graph(Nat.3.suc) = cycle_graph(Nat.4)
    range_set(Nat.3.suc) = range_set(Nat.4)
    is_hamiltonian(cycle_graph(Nat.4), range_set(Nat.4))
}

/// The five-cycle is Hamiltonian.
theorem cycle_graph_5_is_hamiltonian {
    is_hamiltonian(cycle_graph(Nat.5), range_set(Nat.5))
} by {
    cycle_graph_suc_is_hamiltonian(Nat.4)
    Nat.2.suc = Nat.3
    Nat.2 < Nat.2.suc
    Nat.2 < Nat.3
    Nat.3.suc = Nat.4
    Nat.3 < Nat.3.suc
    Nat.3 < Nat.4
    Nat.3 <= Nat.4
    lt_and_lte(Nat.2, Nat.3, Nat.4)
    Nat.2 < Nat.4
    Nat.2 <= Nat.4
    is_hamiltonian(cycle_graph(Nat.4.suc), range_set(Nat.4.suc))
    cycle_graph(Nat.4.suc) = cycle_graph(Nat.5)
    range_set(Nat.4.suc) = range_set(Nat.5)
    is_hamiltonian(cycle_graph(Nat.5), range_set(Nat.5))
}

/// The complete graph on three vertices is Hamiltonian.
theorem complete_graph_3_is_hamiltonian {
    is_hamiltonian(complete_graph[Nat], range_set(Nat.3))
} by {
    complete_graph_suc_is_hamiltonian(Nat.2)
    Nat.2 <= Nat.2
    is_hamiltonian(complete_graph[Nat], range_set(Nat.2.suc))
    range_set(Nat.2.suc) = range_set(Nat.3)
    is_hamiltonian(complete_graph[Nat], range_set(Nat.3))
}

/// The complete graph on four vertices is Hamiltonian.
theorem complete_graph_4_is_hamiltonian {
    is_hamiltonian(complete_graph[Nat], range_set(Nat.4))
} by {
    complete_graph_suc_is_hamiltonian(Nat.3)
    Nat.2.suc = Nat.3
    Nat.2 < Nat.2.suc
    Nat.2 < Nat.3
    Nat.2 <= Nat.3
    is_hamiltonian(complete_graph[Nat], range_set(Nat.3.suc))
    range_set(Nat.3.suc) = range_set(Nat.4)
    is_hamiltonian(complete_graph[Nat], range_set(Nat.4))
}

/// The complete graph on five vertices is Hamiltonian.
theorem complete_graph_5_is_hamiltonian {
    is_hamiltonian(complete_graph[Nat], range_set(Nat.5))
} by {
    complete_graph_suc_is_hamiltonian(Nat.4)
    Nat.2.suc = Nat.3
    Nat.2 < Nat.2.suc
    Nat.2 < Nat.3
    Nat.3.suc = Nat.4
    Nat.3 < Nat.3.suc
    Nat.3 < Nat.4
    Nat.3 <= Nat.4
    lt_and_lte(Nat.2, Nat.3, Nat.4)
    Nat.2 < Nat.4
    Nat.2 <= Nat.4
    is_hamiltonian(complete_graph[Nat], range_set(Nat.4.suc))
    range_set(Nat.4.suc) = range_set(Nat.5)
    is_hamiltonian(complete_graph[Nat], range_set(Nat.5))
}

/// True when every vertex of the walk's vertex list is reachable from its start.
///
/// The membership-forall inside the reachability lemma is packaged as a named
/// predicate so that the induction's unfolding equations stay free of inlined
/// nested quantifiers, which proof search cannot reduce.
define walk_vertices_reachable_from[V](g: SimpleGraph[V], start: V, steps: List[V]) -> Bool {
    forall(x: V) {
        simple_graph_walk_vertices(start, steps).contains(x) implies simple_graph_reachable(g, start, x)
    }
}

/// Every vertex on a walk is reachable from its start.
///
/// Walking the walk's vertex list from the front, each vertex is reachable from the
/// start by the prefix of the walk. This is the step that turns the covering half of
/// the Hamiltonian condition into a connectivity statement.
theorem walk_vertices_reachable_from_start[V](g: SimpleGraph[V], start: V, finish: V, steps: List[V]) {
    simple_graph_walk(g, start, finish, steps) implies
        forall(x: V) {
            simple_graph_walk_vertices(start, steps).contains(x) implies simple_graph_reachable(g, start, x)
        }
} by {
    define pc(xs: List[V], s: V, t: V) -> Bool {
        simple_graph_walk(g, s, t, xs) implies walk_vertices_reachable_from(g, s, xs)
    }

    define p(xs: List[V]) -> Bool {
        forall(s: V, t: V) {
            pc(xs, s, t)
        }
    }

    forall(s: V, t: V) {
        if simple_graph_walk(g, s, t, List.nil[V]) {
            simple_graph_walk_nil(g, s, t)
            s = t
            forall(x: V) {
                if simple_graph_walk_vertices(s, List.nil[V]).contains(x) {
                    simple_graph_walk_vertices(s, List.nil[V]) = List.cons(s, List.nil[V])
                    List.cons(s, List.nil[V]) = List.singleton(s)
                    singleton_contains_imp_eq(s, x)
                    List.singleton(s).contains(x) implies x = s
                    x = s
                    simple_graph_reachable_refl(g, s)
                    simple_graph_reachable(g, s, s)
                    simple_graph_reachable(g, s, x)
                }
                simple_graph_walk_vertices(s, List.nil[V]).contains(x) implies simple_graph_reachable(g, s, x)
            }
            walk_vertices_reachable_from(g, s, List.nil[V]) = forall(x: V) {
                simple_graph_walk_vertices(s, List.nil[V]).contains(x) implies simple_graph_reachable(g, s, x)
            }
            walk_vertices_reachable_from(g, s, List.nil[V])
        }
        pc(List.nil[V], s, t) = (simple_graph_walk(g, s, t, List.nil[V]) implies
            walk_vertices_reachable_from(g, s, List.nil[V]))
        pc(List.nil[V], s, t)
    }
    p(List.nil[V]) = forall(s0: V, t0: V) {
        pc(List.nil[V], s0, t0)
    }
    p(List.nil[V])

    forall(next: V, tail: List[V]) {
        if p(tail) {
            p(tail) = forall(s0: V, t0: V) {
                pc(tail, s0, t0)
            }
            forall(s: V, t: V) {
                if simple_graph_walk(g, s, t, List.cons(next, tail)) {
                    simple_graph_walk_cons_iff(g, s, t, next, tail)
                    g.adj(s, next)
                    simple_graph_walk(g, next, t, tail)
                    forall(x: V) {
                        if simple_graph_walk_vertices(s, List.cons(next, tail)).contains(x) {
                            simple_graph_walk_vertices(s, List.cons(next, tail)) =
                                List.cons(s, List.cons(next, tail))
                            cons_contains_eq(s, List.cons(next, tail), x)
                            List.cons(s, List.cons(next, tail)).contains(x) =
                                (s = x or List.cons(next, tail).contains(x))
                            if x = s {
                                simple_graph_reachable_refl(g, s)
                                simple_graph_reachable(g, s, s)
                                simple_graph_reachable(g, s, x)
                            }
                            if x != s {
                                List.cons(next, tail).contains(x)
                                pc(tail, next, t) = (simple_graph_walk(g, next, t, tail) implies
                                    walk_vertices_reachable_from(g, next, tail))
                                pc(tail, next, t)
                                walk_vertices_reachable_from(g, next, tail)
                                walk_vertices_reachable_from(g, next, tail) = forall(y: V) {
                                    simple_graph_walk_vertices(next, tail).contains(y) implies simple_graph_reachable(g, next, y)
                                }
                                forall(y: V) {
                                    simple_graph_walk_vertices(next, tail).contains(y) implies simple_graph_reachable(g, next, y)
                                }
                                simple_graph_walk_vertices(next, tail).contains(x) implies simple_graph_reachable(g, next, x)
                                simple_graph_walk_vertices(next, tail) = List.cons(next, tail)
                                simple_graph_reachable(g, next, x)
                                simple_graph_reachable_of_adj(g, s, next)
                                g.adj(s, next)
                                simple_graph_reachable(g, s, next)
                                simple_graph_reachable_transitive(g, s, next, x)
                                simple_graph_reachable(g, s, x)
                            }
                            simple_graph_reachable(g, s, x)
                        }
                        simple_graph_walk_vertices(s, List.cons(next, tail)).contains(x) implies simple_graph_reachable(g, s, x)
                    }
                }
                pc(List.cons(next, tail), s, t) = (simple_graph_walk(g, s, t, List.cons(next, tail)) implies
                    walk_vertices_reachable_from(g, s, List.cons(next, tail)))
                pc(List.cons(next, tail), s, t)
            }
            p(List.cons(next, tail))
        }
    }
    forall(next: V, tail: List[V]) { p(tail) implies p(List.cons(next, tail)) }
    list_induction_elim(p, steps)
    p(steps)
    p(steps) = forall(s0: V, t0: V) {
        pc(steps, s0, t0)
    }
    pc(steps, start, finish)
    pc(steps, start, finish) = (simple_graph_walk(g, start, finish, steps) implies
        walk_vertices_reachable_from(g, start, steps))
    if simple_graph_walk(g, start, finish, steps) {
        walk_vertices_reachable_from(g, start, steps)
        walk_vertices_reachable_from(g, start, steps) = forall(x: V) {
            simple_graph_walk_vertices(start, steps).contains(x) implies simple_graph_reachable(g, start, x)
        }
        forall(x: V) {
            simple_graph_walk_vertices(start, steps).contains(x) implies simple_graph_reachable(g, start, x)
        }
    }
}

/// A Hamiltonian graph is connected on its vertex set.
///
/// The Hamiltonian cycle visits every vertex of `s`, and walking the cycle forward
/// from its start reaches each of them; reachability is symmetric and transitive, so
/// any two vertices of `s` are mutually reachable. This is the connectivity half of
/// the statement that a Hamiltonian graph is 2-connected; the full 2-connected form
/// (connected after deleting any single vertex) is stated below but not yet proved.
theorem hamiltonian_imp_set_connected[V](g: SimpleGraph[V], s: FiniteSet[V]) {
    is_hamiltonian(g, s) implies simple_graph_fs_connected(g, s)
} by {
    if is_hamiltonian(g, s) {
        is_hamiltonian(g, s) = exists(start: V, steps: List[V]) {
            is_hamiltonian_cycle(g, s, start, steps)
        }
        exists(start: V, steps: List[V]) {
            is_hamiltonian_cycle(g, s, start, steps)
        }
        let (start: V, steps: List[V]) satisfy {
            is_hamiltonian_cycle(g, s, start, steps)
        }
        is_hamiltonian_cycle(g, s, start, steps) = (
            simple_graph_closed_walk(g, start, steps) and Nat.3 <= steps.length and
            steps.is_unique and walk_vertices_in_set(s, start, steps) and
            walk_visits_set(s, start, steps)
        )
        simple_graph_closed_walk(g, start, steps)
        walk_visits_set(s, start, steps)
        simple_graph_closed_walk_is_walk(g, start, steps)
        simple_graph_walk(g, start, start, steps)
        walk_vertices_reachable_from_start(g, start, start, steps)
        forall(x: V) {
            simple_graph_walk_vertices(start, steps).contains(x) implies simple_graph_reachable(g, start, x)
        }
        walk_visits_set(s, start, steps) = forall(x: V) {
            s.contains(x) implies simple_graph_walk_vertices(start, steps).contains(x)
        }
        forall(x: V, y: V) {
            if s.contains(x) and s.contains(y) {
                s.contains(x) implies simple_graph_walk_vertices(start, steps).contains(x)
                simple_graph_walk_vertices(start, steps).contains(x)
                simple_graph_walk_vertices(start, steps).contains(x) implies simple_graph_reachable(g, start, x)
                simple_graph_reachable(g, start, x)
                s.contains(y) implies simple_graph_walk_vertices(start, steps).contains(y)
                simple_graph_walk_vertices(start, steps).contains(y)
                simple_graph_walk_vertices(start, steps).contains(y) implies simple_graph_reachable(g, start, y)
                simple_graph_reachable(g, start, y)
                simple_graph_reachable_symmetric(g, start, x)
                simple_graph_reachable(g, x, start)
                simple_graph_reachable_transitive(g, x, start, y)
                simple_graph_reachable(g, x, y)
            }
            s.contains(x) and s.contains(y) implies simple_graph_reachable(g, x, y)
        }
        simple_graph_fs_connected(g, s) = forall(x: V, y: V) {
            s.contains(x) and s.contains(y) implies simple_graph_reachable(g, x, y)
        }
        simple_graph_fs_connected(g, s)
    }
}


// ---------------------------------------------------------------------------
// The Petersen graph.
//
// The graph is defined on the naturals with the vertex set cut out by `range_set`,
// exactly as the cycle and path graphs are. The fifteen undirected edges are listed
// once as representatives and adjacency checks both orientations, so symmetry is
// immediate; the distinctness conjunct makes irreflexivity immediate as well, and
// the usual drawing is the outer 5-cycle 0-1-2-3-4-0, the spokes 0-5, 1-6, 2-7,
// 3-8, 4-9, and the inner pentagram 5-7-9-6-8-5.
// ---------------------------------------------------------------------------

/// One representative per undirected edge of the Petersen graph.
///
/// Adjacency checks both orders of every pair, which makes the symmetry of the
/// relation definitional rather than a proof obligation.
let petersen_edge_pairs: List[Pair[Nat, Nat]] = List.cons(Pair.new(Nat.0, Nat.1),
    List.cons(Pair.new(Nat.1, Nat.2),
    List.cons(Pair.new(Nat.2, Nat.3),
    List.cons(Pair.new(Nat.3, Nat.4),
    List.cons(Pair.new(Nat.4, Nat.0),
    List.cons(Pair.new(Nat.0, Nat.5),
    List.cons(Pair.new(Nat.1, Nat.6),
    List.cons(Pair.new(Nat.2, Nat.7),
    List.cons(Pair.new(Nat.3, Nat.8),
    List.cons(Pair.new(Nat.4, Nat.9),
    List.cons(Pair.new(Nat.5, Nat.7),
    List.cons(Pair.new(Nat.7, Nat.9),
    List.cons(Pair.new(Nat.9, Nat.6),
    List.cons(Pair.new(Nat.6, Nat.8),
    List.cons(Pair.new(Nat.8, Nat.5),
    List.nil[Pair[Nat, Nat]])))))))))))))))

/// Adjacency of the Petersen graph.
///
/// Two vertices are adjacent when they are distinct and their unordered pair is one
/// of the fifteen edges.
define petersen_adj(x: Nat, y: Nat) -> Bool {
    x != y and (petersen_edge_pairs.contains(Pair.new(x, y)) or
        petersen_edge_pairs.contains(Pair.new(y, x)))
}

/// Petersen adjacency is symmetric.
theorem petersen_adj_is_symmetric {
    is_symmetric(petersen_adj)
} by {
    forall(x: Nat, y: Nat) {
        if petersen_adj(x, y) {
            petersen_adj(x, y) = (x != y and (petersen_edge_pairs.contains(Pair.new(x, y)) or
                petersen_edge_pairs.contains(Pair.new(y, x))))
            x != y
            petersen_edge_pairs.contains(Pair.new(x, y)) or petersen_edge_pairs.contains(Pair.new(y, x))
            y != x
            petersen_adj(y, x) = (y != x and (petersen_edge_pairs.contains(Pair.new(y, x)) or
                petersen_edge_pairs.contains(Pair.new(x, y))))
            petersen_adj(y, x)
        }
        petersen_adj(x, y) implies petersen_adj(y, x)
    }
    (is_symmetric(petersen_adj) = forall(x: Nat, y: Nat) {
        petersen_adj(x, y) implies petersen_adj(y, x)
    })
    is_symmetric(petersen_adj)
}

/// Petersen adjacency has no loops.
theorem petersen_adj_is_irreflexive {
    is_irreflexive(petersen_adj)
} by {
    forall(x: Nat) {
        if petersen_adj(x, x) {
            petersen_adj(x, x) = (x != x and (petersen_edge_pairs.contains(Pair.new(x, x)) or
                petersen_edge_pairs.contains(Pair.new(x, x))))
            x != x
            false
        }
        not petersen_adj(x, x)
    }
    (is_irreflexive(petersen_adj) = forall(x: Nat) { not petersen_adj(x, x) })
    is_irreflexive(petersen_adj)
}

/// `petersen_adj` satisfies the simple-graph constraint.
theorem petersen_adj_constraint {
    is_symmetric(petersen_adj) and is_irreflexive(petersen_adj)
} by {
    petersen_adj_is_symmetric
    petersen_adj_is_irreflexive
}

/// The Petersen graph exists as a simple graph.
theorem petersen_graph_exists {
    exists(g: SimpleGraph[Nat]) { SimpleGraph.new(petersen_adj) = Option.some(g) }
} by {
    petersen_adj_constraint
}

/// The Petersen graph: ten vertices, three-regular, girth five.
let petersen_graph: SimpleGraph[Nat] satisfy {
    SimpleGraph.new(petersen_adj) = Option.some(petersen_graph)
}

/// Adjacency in the Petersen graph is distinctness plus a listed edge.
theorem petersen_graph_adj_iff(x: Nat, y: Nat) {
    petersen_graph.adj(x, y) = (x != y and (petersen_edge_pairs.contains(Pair.new(x, y)) or
        petersen_edge_pairs.contains(Pair.new(y, x))))
} by {
    petersen_adj_constraint
    SimpleGraph.new(petersen_adj) = Option.some(petersen_graph)
    petersen_graph.adj = petersen_adj
    petersen_graph.adj(x, y) = petersen_adj(x, y)
    (petersen_adj(x, y) = (x != y and (petersen_edge_pairs.contains(Pair.new(x, y)) or
        petersen_edge_pairs.contains(Pair.new(y, x)))))
}

/// The vertex set of the Petersen graph.
let petersen_vertices: FiniteSet[Nat] = range_set(Nat.10)

/// The first ten labels are in increasing order.
theorem petersen_labels_lt {
    Nat.0 < Nat.1 and Nat.1 < Nat.2 and Nat.2 < Nat.3 and Nat.3 < Nat.4 and
    Nat.4 < Nat.5 and Nat.5 < Nat.6 and Nat.6 < Nat.7 and Nat.7 < Nat.8 and Nat.8 < Nat.9
} by {
    Nat.0.suc = Nat.1
    Nat.0 < Nat.0.suc
    Nat.0 < Nat.1
    Nat.1.suc = Nat.2
    Nat.1 < Nat.1.suc
    Nat.1 < Nat.2
    Nat.2.suc = Nat.3
    Nat.2 < Nat.2.suc
    Nat.2 < Nat.3
    Nat.3.suc = Nat.4
    Nat.3 < Nat.3.suc
    Nat.3 < Nat.4
    Nat.4.suc = Nat.5
    Nat.4 < Nat.4.suc
    Nat.4 < Nat.5
    Nat.5.suc = Nat.6
    Nat.5 < Nat.5.suc
    Nat.5 < Nat.6
    Nat.6.suc = Nat.7
    Nat.6 < Nat.6.suc
    Nat.6 < Nat.7
    Nat.7.suc = Nat.8
    Nat.7 < Nat.7.suc
    Nat.7 < Nat.8
    Nat.8.suc = Nat.9
    Nat.8 < Nat.8.suc
    Nat.8 < Nat.9
}

/// A smaller label is a different label.
theorem petersen_lt_imp_ne(a: Nat, b: Nat) {
    a < b implies a != b
} by {
    if a < b {
        if a = b {
            a < a
            lt_not_ref(a)
            not (a < a)
            false
        }
        a != b
    }
}

/// The outer cycle closes: vertex `0` is adjacent to vertex `1`.
///
/// This is a sanity check that the edge list actually records edges; the head of
/// the list is the pair `(0, 1)`.
theorem petersen_graph_adj_01 {
    petersen_graph.adj(Nat.0, Nat.1)
} by {
    petersen_graph_adj_iff(Nat.0, Nat.1)
    petersen_graph.adj(Nat.0, Nat.1) = (Nat.0 != Nat.1 and
        (petersen_edge_pairs.contains(Pair.new(Nat.0, Nat.1)) or
            petersen_edge_pairs.contains(Pair.new(Nat.1, Nat.0))))
    petersen_labels_lt
    (Nat.0 < Nat.1 and Nat.1 < Nat.2 and Nat.2 < Nat.3 and Nat.3 < Nat.4 and
        Nat.4 < Nat.5 and Nat.5 < Nat.6 and Nat.6 < Nat.7 and Nat.7 < Nat.8 and Nat.8 < Nat.9)
    Nat.0 < Nat.1
    petersen_lt_imp_ne(Nat.0, Nat.1)
    Nat.0 != Nat.1
    petersen_edge_pairs = List.cons(Pair.new(Nat.0, Nat.1),
        List.cons(Pair.new(Nat.1, Nat.2),
        List.cons(Pair.new(Nat.2, Nat.3),
        List.cons(Pair.new(Nat.3, Nat.4),
        List.cons(Pair.new(Nat.4, Nat.0),
        List.cons(Pair.new(Nat.0, Nat.5),
        List.cons(Pair.new(Nat.1, Nat.6),
        List.cons(Pair.new(Nat.2, Nat.7),
        List.cons(Pair.new(Nat.3, Nat.8),
        List.cons(Pair.new(Nat.4, Nat.9),
        List.cons(Pair.new(Nat.5, Nat.7),
        List.cons(Pair.new(Nat.7, Nat.9),
        List.cons(Pair.new(Nat.9, Nat.6),
        List.cons(Pair.new(Nat.6, Nat.8),
        List.cons(Pair.new(Nat.8, Nat.5),
        List.nil[Pair[Nat, Nat]])))))))))))))))
    cons_contains_head(Pair.new(Nat.0, Nat.1),
        List.cons(Pair.new(Nat.1, Nat.2),
        List.cons(Pair.new(Nat.2, Nat.3),
        List.cons(Pair.new(Nat.3, Nat.4),
        List.cons(Pair.new(Nat.4, Nat.0),
        List.cons(Pair.new(Nat.0, Nat.5),
        List.cons(Pair.new(Nat.1, Nat.6),
        List.cons(Pair.new(Nat.2, Nat.7),
        List.cons(Pair.new(Nat.3, Nat.8),
        List.cons(Pair.new(Nat.4, Nat.9),
        List.cons(Pair.new(Nat.5, Nat.7),
        List.cons(Pair.new(Nat.7, Nat.9),
        List.cons(Pair.new(Nat.9, Nat.6),
        List.cons(Pair.new(Nat.6, Nat.8),
        List.cons(Pair.new(Nat.8, Nat.5),
        List.nil[Pair[Nat, Nat]])))))))))))))))
    petersen_edge_pairs.contains(Pair.new(Nat.0, Nat.1))
    petersen_edge_pairs.contains(Pair.new(Nat.0, Nat.1)) or petersen_edge_pairs.contains(Pair.new(Nat.1, Nat.0))
    Nat.0 != Nat.1 and (petersen_edge_pairs.contains(Pair.new(Nat.0, Nat.1)) or
        petersen_edge_pairs.contains(Pair.new(Nat.1, Nat.0)))
    petersen_graph.adj(Nat.0, Nat.1)
}

// ---------------------------------------------------------------------------
// Statements that are out of reach of the current proof search.
//
// Each is mathematically true and stated in the library's vocabulary, but its
// proof needs machinery the library does not yet carry (constructive extremal
// arguments, vertex-deletion connectivity, or exhaustive case analysis over a
// ten-vertex graph). They are kept here as commented statements with the proof
// strategy sketched, following the convention of `simple_graph_eulerian.ac`.
// ---------------------------------------------------------------------------

// /// A vertex has degree at least two in a Hamiltonian graph.
// ///
// /// On the Hamiltonian cycle each vertex is flanked by two neighbours: the vertex
// /// before it and the vertex after it on the cycle, which are distinct by the
// /// cycle's uniqueness. The two neighbours lie in the vertex set, so the
// /// neighbourhood has cardinality at least two. Formally one would need the
// /// position of `v` in the walk's vertex list and the two incident walk edges at
// /// that position; the machinery for locating a vertex inside a list is not in
// /// the library yet.
// theorem hamiltonian_cycle_degree_at_least_two[V](g: SimpleGraph[V], s: FiniteSet[V], v: V) {
//     is_hamiltonian(g, s) and s.contains(v) implies Nat.2 <= degree(g, s, v)
// }

// /// Dirac's theorem: minimum degree at least half the order forces Hamiltonicity.
// ///
// /// The classic sufficient condition: if `(g, s)` has `n` vertices and every
// /// vertex has degree at least `n / 2`, then `(g, s)` has a Hamiltonian cycle.
// /// The proof is the maximal-path rotation argument: take a longest simple path,
// /// rotate its end segment whenever the end has a neighbour inside it, and show
// /// that when no rotation is possible the path already covers every vertex, whose
// /// high degrees then close it into a cycle. Formalizing the rotation argument
// /// needs a constructive theory of maximal paths and of segment rotation inside
// /// a path, none of which exists in the library yet.
// theorem dirac_theorem[V](g: SimpleGraph[V], s: FiniteSet[V]) {
//     (forall(v: V) {
//         s.contains(v) implies Nat.2 * degree(g, s, v) >= fs_card(s)
//     }) implies is_hamiltonian(g, s)
// }

// /// A Hamiltonian graph is 2-connected.
// ///
// /// Deleting any single vertex from a Hamiltonian graph leaves a connected graph:
// /// the Hamiltonian cycle minus that vertex is a path through every remaining
// /// vertex. The connectivity half is proved in
// /// `hamiltonian_imp_set_connected`; the deletion half needs a notion of walks
// /// staying inside a proper subset of the vertex set (the vertex set with one
// /// element removed), which `simple_graph_reachable` does not provide - it allows
// /// walks through any vertex of the ambient type.
// define is_two_connected[V](g: SimpleGraph[V], s: FiniteSet[V]) -> Bool {
//     forall(v: V) {
//         s.contains(v) implies forall(x: V, y: V) {
//             fs_remove(s, v).contains(x) and fs_remove(s, v).contains(y) implies
//                 exists(steps: List[V]) {
//                     simple_graph_walk(g, x, y, steps) and
//                     walk_vertices_in_set(fs_remove(s, v), x, steps)
//                 }
//         }
//     }
// }
//
// // theorem hamiltonian_imp_two_connected[V](g: SimpleGraph[V], s: FiniteSet[V]) {
// //     is_hamiltonian(g, s) implies is_two_connected(g, s)
// // }

// /// The Petersen graph is not Hamiltonian.
// ///
// /// This is the classical companion to the positive results above: the Petersen
// /// graph has no cycle visiting all ten vertices. The standard proof draws the
// /// graph as an outer 5-cycle, an inner pentagram, and five spokes, and argues
// /// that a Hamiltonian cycle must use an even number of the five spokes - zero,
// /// two, or four - and that each case is impossible: with no spokes the cycle
// /// cannot leave the outer ring, with two spokes the two rings cannot both be
// /// fully covered (an arc of the outer 5-cycle between two spoke endpoints never
// /// has five vertices while the corresponding inner arc is not a path of the
// /// pentagram), and with four spokes the unused spoke forces a repeated vertex.
// /// Formalizing the argument requires counting how many of the walk's consecutive
// /// pairs are spokes and a case analysis over those counts; the ten-vertex search
// /// space is far beyond the current proof search.
// theorem petersen_graph_not_hamiltonian {
//     not is_hamiltonian(petersen_graph, petersen_vertices)
// }
