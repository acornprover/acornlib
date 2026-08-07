from nat import Nat
from finite_set import FiniteSet
from data.finite.finite_set_card import fs_card
from data.finite.finite_set_nonempty import finite_set_has_member_of_positive_card
from graph.simple_graph import SimpleGraph
from graph.simple_graph_degree import degree, neighborhood, neighborhood_contains_eq
from graph.simple_graph_bipartite import has_no_isolated_vertices, has_no_isolated_vertices_apply

numerals Nat

/// A vertex of positive degree has a neighbor in the ambient set.
///
/// Earlier work stated the no-isolated-vertices condition as a witnessing map because
/// there was no way to get from a positive count to an element. Finite-set induction
/// supplies that, so the condition can now be stated through `degree` and the witness
/// recovered when needed.
theorem neighbor_exists_of_positive_degree[V](
    g: SimpleGraph[V], s: FiniteSet[V], v: V
) {
    Nat.0 < degree(g, s, v) implies exists(w: V) { s.contains(w) and g.adj(v, w) }
} by {
    if Nat.0 < degree(g, s, v) {
        degree(g, s, v) = fs_card(neighborhood(g, s, v))
        Nat.0 < fs_card(neighborhood(g, s, v))
        finite_set_has_member_of_positive_card(neighborhood(g, s, v))
        exists(x: V) { neighborhood(g, s, v).contains(x) }
        let (w: V) satisfy {
            neighborhood(g, s, v).contains(w)
        }
        neighborhood_contains_eq(g, s, v, w)
        s.contains(w) and g.adj(v, w)
        exists(u: V) { s.contains(u) and g.adj(v, u) }
    }
}

/// In a graph without isolated vertices, every vertex has a neighbor.
theorem neighbor_exists_of_no_isolated[V](
    g: SimpleGraph[V], s: FiniteSet[V], v: V
) {
    has_no_isolated_vertices(g, s) and s.contains(v) implies exists(w: V) { s.contains(w) and g.adj(v, w) }
} by {
    if has_no_isolated_vertices(g, s) and s.contains(v) {
        has_no_isolated_vertices_apply(g, s, v)
        Nat.0 < degree(g, s, v)
        neighbor_exists_of_positive_degree(g, s, v)
        exists(w: V) { s.contains(w) and g.adj(v, w) }
    }
}

/// A vertex with no neighbor in the ambient set has degree zero.
theorem degree_zero_of_no_neighbor[V](
    g: SimpleGraph[V], s: FiniteSet[V], v: V
) {
    (forall(w: V) { not (s.contains(w) and g.adj(v, w)) }) implies degree(g, s, v) = Nat.0
} by {
    if forall(w: V) { not (s.contains(w) and g.adj(v, w)) } {
        if Nat.0 < degree(g, s, v) {
            neighbor_exists_of_positive_degree(g, s, v)
            let (w: V) satisfy {
                s.contains(w) and g.adj(v, w)
            }
            not (s.contains(w) and g.adj(v, w))
            false
        }
        degree(g, s, v) = Nat.0
    }
}
