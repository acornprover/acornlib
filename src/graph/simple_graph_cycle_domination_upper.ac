from nat import Nat, lte_trans, lte_mul_both, small_mod
from finite_set import FiniteSet
from data.nat.nat_range_set import range_set, range_set_lt
from graph.simple_graph_domination_number import domination_number
from graph.simple_graph_domination_monotone_on import has_all_edges_of_on,
    has_all_edges_of_on_intro, domination_number_antitone_in_edges_on
from graph.simple_graph_path import path_graph, path_graph_adj_iff
from graph.simple_graph_path_domination_upper import path_domination_number_upper_bound
from graph.simple_graph_cycle import cycle_graph, cycle_graph_adj_iff
from graph.simple_graph_cycle_domination import cycle_domination_number_lower_bound

numerals Nat

/// Inside its range, the cycle has every edge the path has.
///
/// Two consecutive naturals below `n` are consecutive modulo `n` as well, and distinct, so
/// they are adjacent in the cycle. The wrap edge is extra; the containment goes only one way.
///
/// It holds only inside the range. The path edge from `n - 1` to `n` has one endpoint outside
/// the cycle, and there the cycle relation is not the intended one, so the unrestricted
/// `has_all_edges_of` is false here.
theorem cycle_has_all_path_edges_on_range(n: Nat) {
    has_all_edges_of_on(cycle_graph(n), path_graph, range_set(n))
} by {
    forall(x: Nat, y: Nat) {
        if range_set(n).contains(x) and range_set(n).contains(y) and path_graph.adj(x, y) {
            range_set_lt(n, x)
            x < n
            range_set_lt(n, y)
            y < n
            path_graph_adj_iff(x, y)
            (path_graph.adj(x, y) = (x.suc = y or y.suc = x))
            if x.suc = y {
                small_mod(y, n)
                y.mod(n) = y
                x.suc.mod(n) = y
                x != y
                (x.suc.mod(n) = y or y.suc.mod(n) = x)
            }
            if y.suc = x {
                small_mod(x, n)
                x.mod(n) = x
                y.suc.mod(n) = x
                x != y
                (x.suc.mod(n) = y or y.suc.mod(n) = x)
            }
            x != y
            (x.suc.mod(n) = y or y.suc.mod(n) = x)
            cycle_graph_adj_iff(n, x, y)
            (cycle_graph(n).adj(x, y)
                = (x != y and (x.suc.mod(n) = y or y.suc.mod(n) = x)))
            cycle_graph(n).adj(x, y)
        }
        (range_set(n).contains(x) and range_set(n).contains(y) and path_graph.adj(x, y)
            implies cycle_graph(n).adj(x, y))
    }
    has_all_edges_of_on_intro(cycle_graph(n), path_graph, range_set(n))
    has_all_edges_of_on(cycle_graph(n), path_graph, range_set(n))
}

/// A cycle is no harder to dominate than the path of the same length.
///
/// The cycle has the path's edges and one more, and domination can only get easier as edges
/// are added.
theorem cycle_domination_number_le_path(n: Nat) {
    domination_number(cycle_graph(n), range_set(n))
        <= domination_number(path_graph, range_set(n))
} by {
    cycle_has_all_path_edges_on_range(n)
    has_all_edges_of_on(cycle_graph(n), path_graph, range_set(n))
    domination_number_antitone_in_edges_on(cycle_graph(n), path_graph, range_set(n))
    (domination_number(cycle_graph(n), range_set(n))
        <= domination_number(path_graph, range_set(n)))
}

/// The domination number of a cycle is at most a third of its length, rounded up.
theorem cycle_domination_number_upper_bound(n: Nat) {
    Nat.3 * domination_number(cycle_graph(n), range_set(n)) <= n + Nat.2
} by {
    cycle_domination_number_le_path(n)
    (domination_number(cycle_graph(n), range_set(n))
        <= domination_number(path_graph, range_set(n)))
    lte_mul_both(Nat.3, domination_number(cycle_graph(n), range_set(n)),
        domination_number(path_graph, range_set(n)))
    (Nat.3 * domination_number(cycle_graph(n), range_set(n))
        <= Nat.3 * domination_number(path_graph, range_set(n)))
    path_domination_number_upper_bound(n)
    (Nat.3 * domination_number(path_graph, range_set(n)) <= n + Nat.2)
    lte_trans(Nat.3 * domination_number(cycle_graph(n), range_set(n)),
        Nat.3 * domination_number(path_graph, range_set(n)), n + Nat.2)
    (Nat.3 * domination_number(cycle_graph(n), range_set(n)) <= n + Nat.2)
}

/// The domination number of a cycle, pinned between two bounds.
///
/// The same pair as for the path, and by the same reckoning: `3 * gamma` lies in
/// `{n, n + 1, n + 2}`, which leaves exactly one value for `gamma`.
theorem cycle_domination_number_bounds(n: Nat) {
    n <= Nat.3 * domination_number(cycle_graph(n), range_set(n))
        and Nat.3 * domination_number(cycle_graph(n), range_set(n)) <= n + Nat.2
} by {
    cycle_domination_number_lower_bound(n)
    n <= Nat.3 * domination_number(cycle_graph(n), range_set(n))
    cycle_domination_number_upper_bound(n)
    (Nat.3 * domination_number(cycle_graph(n), range_set(n)) <= n + Nat.2)
}
