from nat import Nat
from finite_set import FiniteSet, finite_set_subset_contains
from data.finite.finite_set_filter import finite_set_filter, finite_set_filter_contains_eq,
    finite_set_filter_subset
from data.finite.finite_set_subset_intro import fs_subset_eq_intro
from graph.simple_graph import SimpleGraph

numerals Nat

/// True if `w` is a vertex of `s` outside `b` adjacent to `u`.
///
/// In the colouring language of zero forcing, `b` is the set of blue vertices and this says
/// `w` is a white neighbor of `u`. Named because the forcing rule is a uniqueness statement
/// about exactly these vertices.
define is_white_neighbor[V](
    g: SimpleGraph[V], s: FiniteSet[V], b: FiniteSet[V], u: V, w: V
) -> Bool {
    s.contains(w) and not b.contains(w) and g.adj(u, w)
}

/// True if `u` is blue and `v` is its only white neighbor.
///
/// The zero forcing colour-change rule: such a `u` forces `v` to become blue. The rule is
/// deterministic in the sense that `u` has no choice about which vertex it forces.
define can_force[V](
    g: SimpleGraph[V], s: FiniteSet[V], b: FiniteSet[V], u: V, v: V
) -> Bool {
    b.contains(u) and is_white_neighbor(g, s, b, u, v) and forall(w: V) {
        (is_white_neighbor(g, s, b, u, w) implies w = v)
    }
}

/// A forcing vertex is blue, and the forced vertex is one of its white neighbors.
theorem can_force_apply[V](
    g: SimpleGraph[V], s: FiniteSet[V], b: FiniteSet[V], u: V, v: V
) {
    can_force(g, s, b, u, v) implies b.contains(u) and is_white_neighbor(g, s, b, u, v)
} by {
    if can_force(g, s, b, u, v) {
        can_force(g, s, b, u, v) = (b.contains(u) and is_white_neighbor(g, s, b, u, v)
            and forall(w: V) {
                (is_white_neighbor(g, s, b, u, w) implies w = v)
            })
        b.contains(u) and is_white_neighbor(g, s, b, u, v)
    }
}

/// The forced vertex is the only white neighbor.
theorem can_force_unique[V](
    g: SimpleGraph[V], s: FiniteSet[V], b: FiniteSet[V], u: V, v: V, w: V
) {
    can_force(g, s, b, u, v) and is_white_neighbor(g, s, b, u, w) implies w = v
} by {
    if can_force(g, s, b, u, v) and is_white_neighbor(g, s, b, u, w) {
        can_force(g, s, b, u, v) = (b.contains(u) and is_white_neighbor(g, s, b, u, v)
            and forall(x: V) {
                (is_white_neighbor(g, s, b, u, x) implies x = v)
            })
        forall(x: V) {
            (is_white_neighbor(g, s, b, u, x) implies x = v)
        }
        (is_white_neighbor(g, s, b, u, w) implies w = v)
        w = v
    }
}

/// The two conditions give a forcing pair.
theorem can_force_intro[V](
    g: SimpleGraph[V], s: FiniteSet[V], b: FiniteSet[V], u: V, v: V
) {
    b.contains(u) and is_white_neighbor(g, s, b, u, v) and (forall(w: V) {
        (is_white_neighbor(g, s, b, u, w) implies w = v)
    }) implies can_force(g, s, b, u, v)
} by {
    if b.contains(u) and is_white_neighbor(g, s, b, u, v) and forall(w: V) {
        (is_white_neighbor(g, s, b, u, w) implies w = v)
    } {
        can_force(g, s, b, u, v) = (b.contains(u) and is_white_neighbor(g, s, b, u, v)
            and forall(x: V) {
                (is_white_neighbor(g, s, b, u, x) implies x = v)
            })
        can_force(g, s, b, u, v)
    }
}

/// A vertex forces at most one other.
///
/// The rule is deterministic: if `u` can force both `v` and `v2`, both are its unique white
/// neighbor, so they coincide.
theorem can_force_deterministic[V](
    g: SimpleGraph[V], s: FiniteSet[V], b: FiniteSet[V], u: V, v: V, v2: V
) {
    can_force(g, s, b, u, v) and can_force(g, s, b, u, v2) implies v = v2
} by {
    if can_force(g, s, b, u, v) and can_force(g, s, b, u, v2) {
        can_force_apply(g, s, b, u, v2)
        is_white_neighbor(g, s, b, u, v2)
        can_force_unique(g, s, b, u, v, v2)
        v2 = v
    }
}

/// True if some blue vertex forces `v`.
define is_forceable[V](
    g: SimpleGraph[V], s: FiniteSet[V], b: FiniteSet[V], v: V
) -> Bool {
    exists(u: V) {
        can_force(g, s, b, u, v)
    }
}

/// A forcing vertex witnesses forceability.
theorem is_forceable_intro[V](
    g: SimpleGraph[V], s: FiniteSet[V], b: FiniteSet[V], u: V, v: V
) {
    can_force(g, s, b, u, v) implies is_forceable(g, s, b, v)
} by {
    if can_force(g, s, b, u, v) {
        is_forceable(g, s, b, v) = exists(x: V) {
            can_force(g, s, b, x, v)
        }
        exists(x: V) {
            can_force(g, s, b, x, v)
        }
        is_forceable(g, s, b, v)
    }
}

/// A forcing vertex can be extracted.
theorem is_forceable_witness[V](
    g: SimpleGraph[V], s: FiniteSet[V], b: FiniteSet[V], v: V
) {
    is_forceable(g, s, b, v) implies exists(u: V) { can_force(g, s, b, u, v) }
} by {
    if is_forceable(g, s, b, v) {
        is_forceable(g, s, b, v) = exists(x: V) {
            can_force(g, s, b, x, v)
        }
        exists(x: V) {
            can_force(g, s, b, x, v)
        }
    }
}

/// A forceable vertex lies in the ambient set and is not yet blue.
theorem is_forceable_white[V](
    g: SimpleGraph[V], s: FiniteSet[V], b: FiniteSet[V], v: V
) {
    is_forceable(g, s, b, v) implies s.contains(v) and not b.contains(v)
} by {
    if is_forceable(g, s, b, v) {
        is_forceable_witness(g, s, b, v)
        let (u: V) satisfy {
            can_force(g, s, b, u, v)
        }
        can_force_apply(g, s, b, u, v)
        is_white_neighbor(g, s, b, u, v)
        is_white_neighbor(g, s, b, u, v) =
            (s.contains(v) and not b.contains(v) and g.adj(u, v))
        s.contains(v) and not b.contains(v)
    }
}

/// The vertices that are blue or become blue in one round.
///
/// Packaged as a predicate so the derived set below is a single filter of the ambient set.
define derived_pred[V](
    g: SimpleGraph[V], s: FiniteSet[V], b: FiniteSet[V]
) -> (V -> Bool) {
    function(v: V) {
        b.contains(v) or is_forceable(g, s, b, v)
    }
}

/// The set of blue vertices after one round of forcing.
///
/// Every vertex that can be forced is forced simultaneously. Applying the rule one vertex at
/// a time reaches the same final colouring, so nothing is lost by taking the whole round at
/// once, and the round is a function of the current colouring alone.
define derived_set[V](
    g: SimpleGraph[V], s: FiniteSet[V], b: FiniteSet[V]
) -> FiniteSet[V] {
    finite_set_filter(s, derived_pred(g, s, b))
}

/// Membership in the derived set.
theorem derived_set_contains_eq[V](
    g: SimpleGraph[V], s: FiniteSet[V], b: FiniteSet[V], v: V
) {
    derived_set(g, s, b).contains(v) =
        (s.contains(v) and (b.contains(v) or is_forceable(g, s, b, v)))
} by {
    finite_set_filter_contains_eq(s, derived_pred(g, s, b), v)
    derived_pred(g, s, b)(v) = (b.contains(v) or is_forceable(g, s, b, v))
}

/// The derived set sits inside the ambient set.
theorem derived_set_subset[V](g: SimpleGraph[V], s: FiniteSet[V], b: FiniteSet[V]) {
    derived_set(g, s, b).subset_eq(s)
} by {
    finite_set_filter_subset(s, derived_pred(g, s, b))
}

/// Forcing never uncolours a vertex.
theorem derived_set_contains_blue[V](
    g: SimpleGraph[V], s: FiniteSet[V], b: FiniteSet[V], v: V
) {
    b.subset_eq(s) and b.contains(v) implies derived_set(g, s, b).contains(v)
} by {
    if b.subset_eq(s) and b.contains(v) {
        finite_set_subset_contains(b, s, v)
        s.contains(v)
        derived_set_contains_eq(g, s, b, v)
        derived_set(g, s, b).contains(v)
    }
}

/// The blue set grows under one round of forcing.
theorem derived_set_grows[V](g: SimpleGraph[V], s: FiniteSet[V], b: FiniteSet[V]) {
    b.subset_eq(s) implies b.subset_eq(derived_set(g, s, b))
} by {
    if b.subset_eq(s) {
        forall(v: V) {
            if b.contains(v) {
                derived_set_contains_blue(g, s, b, v)
                derived_set(g, s, b).contains(v)
            }
            (b.contains(v) implies derived_set(g, s, b).contains(v))
        }
        fs_subset_eq_intro(b, derived_set(g, s, b))
        b.subset_eq(derived_set(g, s, b))
    }
}

/// A forced vertex is blue after the round.
theorem derived_set_contains_forced[V](
    g: SimpleGraph[V], s: FiniteSet[V], b: FiniteSet[V], u: V, v: V
) {
    can_force(g, s, b, u, v) implies derived_set(g, s, b).contains(v)
} by {
    if can_force(g, s, b, u, v) {
        is_forceable_intro(g, s, b, u, v)
        is_forceable(g, s, b, v)
        is_forceable_white(g, s, b, v)
        s.contains(v)
        derived_set_contains_eq(g, s, b, v)
        derived_set(g, s, b).contains(v)
    }
}

/// True if no vertex can be forced.
///
/// A colouring where the rule has nothing left to do. The zero forcing closure of a set is
/// the closed colouring it reaches.
define is_forcing_closed[V](
    g: SimpleGraph[V], s: FiniteSet[V], b: FiniteSet[V]
) -> Bool {
    forall(v: V) {
        not is_forceable(g, s, b, v)
    }
}

/// Nothing is forceable in a closed colouring.
theorem is_forcing_closed_apply[V](
    g: SimpleGraph[V], s: FiniteSet[V], b: FiniteSet[V], v: V
) {
    is_forcing_closed(g, s, b) implies not is_forceable(g, s, b, v)
} by {
    if is_forcing_closed(g, s, b) {
        is_forcing_closed(g, s, b) = forall(x: V) {
            not is_forceable(g, s, b, x)
        }
        forall(x: V) {
            not is_forceable(g, s, b, x)
        }
        not is_forceable(g, s, b, v)
    }
}

/// A pointwise absence of forceable vertices is closedness.
theorem is_forcing_closed_intro[V](
    g: SimpleGraph[V], s: FiniteSet[V], b: FiniteSet[V]
) {
    (forall(v: V) { not is_forceable(g, s, b, v) }) implies is_forcing_closed(g, s, b)
} by {
    if forall(v: V) { not is_forceable(g, s, b, v) } {
        is_forcing_closed(g, s, b) = forall(x: V) {
            not is_forceable(g, s, b, x)
        }
        is_forcing_closed(g, s, b)
    }
}

/// A round of forcing changes nothing exactly when the colouring is closed.
///
/// This is what makes the closure detectable: the iteration below has finished as soon as one
/// round is a fixed point.
theorem derived_set_eq_of_closed[V](
    g: SimpleGraph[V], s: FiniteSet[V], b: FiniteSet[V], v: V
) {
    b.subset_eq(s) and is_forcing_closed(g, s, b)
        implies derived_set(g, s, b).contains(v) = b.contains(v)
} by {
    if b.subset_eq(s) and is_forcing_closed(g, s, b) {
        if derived_set(g, s, b).contains(v) {
            derived_set_contains_eq(g, s, b, v)
            s.contains(v) and (b.contains(v) or is_forceable(g, s, b, v))
            if not b.contains(v) {
                is_forceable(g, s, b, v)
                is_forcing_closed_apply(g, s, b, v)
                not is_forceable(g, s, b, v)
                false
            }
            b.contains(v)
        }
        if b.contains(v) {
            derived_set_contains_blue(g, s, b, v)
            derived_set(g, s, b).contains(v)
        }
        (derived_set(g, s, b).contains(v) implies b.contains(v))
        (b.contains(v) implies derived_set(g, s, b).contains(v))
        derived_set(g, s, b).contains(v) = b.contains(v)
    }
}

/// The whole ambient set is closed.
///
/// There is nothing white left, so no vertex has a white neighbor to force.
theorem ambient_is_forcing_closed[V](g: SimpleGraph[V], s: FiniteSet[V]) {
    is_forcing_closed(g, s, s)
} by {
    forall(v: V) {
        if is_forceable(g, s, s, v) {
            is_forceable_white(g, s, s, v)
            s.contains(v)
            not s.contains(v)
            false
        }
        not is_forceable(g, s, s, v)
    }
    is_forcing_closed_intro(g, s, s)
    is_forcing_closed(g, s, s)
}
