from nat import Nat, is_min, has_min, is_min_apply, is_min_false_below, false_below,
    false_below_apply, lte_antisymm, lte_trans
from finite_set import FiniteSet
from data.finite.finite_set_card import fs_card
from graph.simple_graph import SimpleGraph
from graph.simple_graph_domination import is_dominating_set
from graph.simple_graph_domination_number import domination_number, domination_number_is_least
from graph.simple_graph_independent_domination import is_independent_subset,
    is_independent_subset_apply, is_independent_dominating_set,
    is_independent_dominating_set_apply, is_independent_dominating_set_intro,
    domination_number_le_independent_dominating
from graph.simple_graph_max_independent import independent_dominating_set_exists,
    independence_number, independence_number_is_greatest

numerals Nat

/// True of the sizes achieved by independent dominating subsets of `s`.
///
/// Packaged as a predicate on `Nat` so the minimum construction applies to it, in the same
/// shape as `dominating_size_pred` and `zero_forcing_size_pred`.
define independent_dominating_size_pred[V](
    g: SimpleGraph[V], s: FiniteSet[V]
) -> (Nat -> Bool) {
    function(n: Nat) {
        exists(t: FiniteSet[V]) {
            is_independent_dominating_set(g, s, t) and fs_card(t) = n
        }
    }
}

/// An independent dominating set achieves its own size.
theorem independent_dominating_size_pred_intro[V](
    g: SimpleGraph[V], s: FiniteSet[V], t: FiniteSet[V]
) {
    is_independent_dominating_set(g, s, t)
        implies independent_dominating_size_pred(g, s)(fs_card(t))
} by {
    if is_independent_dominating_set(g, s, t) {
        independent_dominating_size_pred(g, s)(fs_card(t)) = exists(r: FiniteSet[V]) {
            is_independent_dominating_set(g, s, r) and fs_card(r) = fs_card(t)
        }
        exists(r: FiniteSet[V]) {
            is_independent_dominating_set(g, s, r) and fs_card(r) = fs_card(t)
        }
        independent_dominating_size_pred(g, s)(fs_card(t))
    }
}

/// Some size is achieved.
///
/// Every vertex set has a maximal independent subset, and a maximal independent set dominates,
/// so the family is never empty. This is what makes the minimum below well defined, and it is
/// the reason the independence number had to come first.
theorem independent_dominating_size_pred_witness[V](g: SimpleGraph[V], s: FiniteSet[V]) {
    exists(n: Nat) { independent_dominating_size_pred(g, s)(n) }
} by {
    independent_dominating_set_exists(g, s)
    let (t: FiniteSet[V]) satisfy {
        is_independent_dominating_set(g, s, t)
    }
    independent_dominating_size_pred_intro(g, s, t)
    independent_dominating_size_pred(g, s)(fs_card(t))
    exists(n: Nat) { independent_dominating_size_pred(g, s)(n) }
}

/// The fewest vertices in an independent dominating subset of `s`.
///
/// The independent domination number. It sits above the domination number, since an
/// independent dominating set is in particular a dominating one, and the constraint of
/// independence can only push the minimum up.
let independent_domination_number[V](g: SimpleGraph[V], s: FiniteSet[V]) -> result: Nat satisfy {
    is_min(independent_dominating_size_pred(g, s), result)
} by {
    independent_dominating_size_pred_witness(g, s)
    let (n: Nat) satisfy {
        independent_dominating_size_pred(g, s)(n)
    }
    has_min(independent_dominating_size_pred(g, s), n)
}

/// Some independent dominating set attains the independent domination number.
theorem independent_domination_number_attained[V](g: SimpleGraph[V], s: FiniteSet[V]) {
    independent_dominating_size_pred(g, s)(independent_domination_number(g, s))
} by {
    is_min(independent_dominating_size_pred(g, s), independent_domination_number(g, s))
    is_min_apply(independent_dominating_size_pred(g, s), independent_domination_number(g, s))
}

/// No independent dominating set is smaller than the independent domination number.
theorem independent_domination_number_is_least[V](
    g: SimpleGraph[V], s: FiniteSet[V], t: FiniteSet[V]
) {
    is_independent_dominating_set(g, s, t)
        implies independent_domination_number(g, s) <= fs_card(t)
} by {
    if is_independent_dominating_set(g, s, t) {
        independent_dominating_size_pred_intro(g, s, t)
        independent_dominating_size_pred(g, s)(fs_card(t))
        is_min(independent_dominating_size_pred(g, s), independent_domination_number(g, s))
        is_min_false_below(independent_dominating_size_pred(g, s),
            independent_domination_number(g, s))
        false_below(independent_dominating_size_pred(g, s),
            independent_domination_number(g, s))
        if fs_card(t) < independent_domination_number(g, s) {
            false_below_apply(independent_dominating_size_pred(g, s),
                independent_domination_number(g, s), fs_card(t))
            not independent_dominating_size_pred(g, s)(fs_card(t))
            false
        }
        independent_domination_number(g, s) <= fs_card(t)
    }
}

/// An independent dominating set attaining the minimum exists.
theorem minimum_independent_dominating_set_exists[V](g: SimpleGraph[V], s: FiniteSet[V]) {
    exists(t: FiniteSet[V]) {
        is_independent_dominating_set(g, s, t)
            and fs_card(t) = independent_domination_number(g, s)
    }
} by {
    independent_domination_number_attained(g, s)
    independent_dominating_size_pred(g, s)(independent_domination_number(g, s))
    (independent_dominating_size_pred(g, s)(independent_domination_number(g, s))
        = exists(t: FiniteSet[V]) {
            is_independent_dominating_set(g, s, t)
                and fs_card(t) = independent_domination_number(g, s)
        })
    exists(t: FiniteSet[V]) {
        is_independent_dominating_set(g, s, t)
            and fs_card(t) = independent_domination_number(g, s)
    }
}

/// The domination number is at most the independent domination number.
///
/// The classical inequality. Requiring the dominating set to be independent as well can only
/// rule out candidates, so the minimum over the smaller family is no smaller.
theorem domination_number_le_independent_domination_number[V](
    g: SimpleGraph[V], s: FiniteSet[V]
) {
    domination_number(g, s) <= independent_domination_number(g, s)
} by {
    minimum_independent_dominating_set_exists(g, s)
    let (t: FiniteSet[V]) satisfy {
        is_independent_dominating_set(g, s, t)
            and fs_card(t) = independent_domination_number(g, s)
    }
    domination_number_le_independent_dominating(g, s, t)
    domination_number(g, s) <= fs_card(t)
    domination_number(g, s) <= independent_domination_number(g, s)
}

/// The independent domination number is at most the independence number.
///
/// An independent dominating set is in particular an independent subset, so its size is
/// bounded by the largest such, while the minimum over independent dominating sets is bounded
/// by that one set. Together with the previous theorem this gives the classical chain
/// `gamma(G) <= i(G) <= alpha(G)`.
theorem independent_domination_number_le_independence_number[V](
    g: SimpleGraph[V], s: FiniteSet[V]
) {
    independent_domination_number(g, s) <= independence_number(g, s)
} by {
    minimum_independent_dominating_set_exists(g, s)
    let (t: FiniteSet[V]) satisfy {
        is_independent_dominating_set(g, s, t)
            and fs_card(t) = independent_domination_number(g, s)
    }
    is_independent_dominating_set_apply(g, s, t)
    is_independent_subset(g, s, t)
    independence_number_is_greatest(g, s, t)
    fs_card(t) <= independence_number(g, s)
    independent_domination_number(g, s) <= independence_number(g, s)
}

/// The domination number is at most the independence number.
///
/// The two ends of the chain.
theorem domination_number_le_independence_number[V](g: SimpleGraph[V], s: FiniteSet[V]) {
    domination_number(g, s) <= independence_number(g, s)
} by {
    domination_number_le_independent_domination_number(g, s)
    domination_number(g, s) <= independent_domination_number(g, s)
    independent_domination_number_le_independence_number(g, s)
    independent_domination_number(g, s) <= independence_number(g, s)
    lte_trans(domination_number(g, s), independent_domination_number(g, s),
        independence_number(g, s))
    domination_number(g, s) <= independence_number(g, s)
}
