/// Deeper vertex-colouring results: the trivial `|V|` bound, the odd-cycle
/// characterisation of two-colourability, two-colourability of trees, the greedy
/// bound (statement), and the five-colour theorem (statement).

from nat import Nat, lte_antisymm, lt_or_lte, not_lt_zero, lt_and_lte, lte_and_lt,
    lt_imp_lte_suc, lt_not_ref, is_min, has_min, is_min_apply, is_min_false_below,
    false_below, false_below_apply, pos_of_ne_zero, lt_add_left, add_assoc, add_comm
from list import List, add_length, cons_unique_of_tail_unique_not_contains, tail_cancels_cons,
    drop_one
from data.list.list_cons_membership import cons_contains_eq, cons_contains_head,
    cons_contains_of_tail_contains, nil_not_contains
from data.basic.logic import not_exists_forward
from finite_set import FiniteSet
from data.finite.finite_set_card import fs_card
from graph.simple_graph import SimpleGraph, induced_subgraph, induced_subgraph_adj_eq,
    simple_graph_adj_irreflexive
from graph.simple_graph_chromatic import is_k_colorable, is_k_colorable_intro,
    is_k_colorable_of_card, is_k_colorable_two_imp_is_bipartite,
    is_bipartite_imp_is_k_colorable_two, chromatic_number, is_chromatic_number,
    is_chromatic_number_not_k_colorable_below, chromatic_number_is_chromatic_number,
    is_proper_coloring_on, is_proper_coloring_on_intro, nat_lt_three_cases, nat_suc_eq_cancel
from graph.simple_graph_bipartite import is_bipartite
from graph.simple_graph_bipartite_odd_cycle import nat_odd, nat_odd_add, nat_odd_ne_zero,
    walk_in_set, walk_in_set_nil, walk_in_set_cons, walk_in_set_left, walk_in_set_right,
    walk_in_set_concat, simple_graph_walk_concat, has_odd_cycle, is_bipartite_no_odd_cycle,
    has_odd_closed_walk, is_bipartite_of_no_odd_closed_walk, walk_reachable_from
from graph.simple_graph_walks import simple_graph_walk, simple_graph_walk_nil,
    simple_graph_walk_cons_iff, simple_graph_closed_walk, simple_graph_closed_walk_intro,
    simple_graph_closed_walk_is_walk, simple_graph_reachable_has_walk
from graph.simple_graph_tree import is_tree, tree_acyclic, tree_connected_on, is_acyclic,
    is_acyclic_cycle_contradiction, simple_graph_cycle, simple_graph_connected_on
from graph.simple_graph_connectivity import simple_graph_reachable
from graph.simple_graph_forest import length_zero_imp_nil, drop_cons, find_first_idx_cons
from graph.planar_euler import nat_const_fn

numerals Nat

// ============================================================================
// The trivial bound: chi(G) <= |V|.  A graph on a finite vertex set is coloured by
// one colour per vertex, so the chromatic number is at most the number of vertices.
// ============================================================================

/// Every graph on a finite vertex set is colourable with one colour per vertex: colour
/// each vertex by its position in an enumeration of the vertex set.
theorem is_k_colorable_card_bound[V](g: SimpleGraph[V], s: FiniteSet[V]) {
    is_k_colorable(g, s, fs_card(s))
} by {
    is_k_colorable_of_card(g, s)
    is_k_colorable(g, s, fs_card(s))
}

/// The chromatic number of a graph is at most the number of vertices.
theorem chromatic_number_le_card[V](g: SimpleGraph[V], s: FiniteSet[V]) {
    chromatic_number(g, s) <= fs_card(s)
} by {
    chromatic_number_is_chromatic_number(g, s)
    is_chromatic_number(g, s, chromatic_number(g, s))
    if fs_card(s) < chromatic_number(g, s) {
        is_chromatic_number_not_k_colorable_below(g, s, chromatic_number(g, s), fs_card(s))
        not is_k_colorable(g, s, fs_card(s))
        is_k_colorable_card_bound(g, s)
        is_k_colorable(g, s, fs_card(s))
        false
    }
    not (fs_card(s) < chromatic_number(g, s))
    lt_or_lte(fs_card(s), chromatic_number(g, s))
    fs_card(s) < chromatic_number(g, s) or chromatic_number(g, s) <= fs_card(s)
    chromatic_number(g, s) <= fs_card(s)
}

// ============================================================================
// Two-colourability and odd cycles, forward direction: a two-colourable graph has
// no odd cycle.  A two-colouring is a bipartition, and every closed walk of a
// bipartite graph has even length.
// ============================================================================

/// A two-colourable graph has no odd cycle.
theorem is_k_colorable_two_imp_no_odd_cycle[V](g: SimpleGraph[V], s: FiniteSet[V]) {
    is_k_colorable(g, s, Nat.2) implies not has_odd_cycle(g, s)
} by {
    if is_k_colorable(g, s, Nat.2) {
        is_k_colorable_two_imp_is_bipartite(g, s)
        is_bipartite(g, s)
        is_bipartite_no_odd_cycle(g, s)
        not has_odd_cycle(g, s)
    }
}

// ============================================================================
// Odd closed walks contain odd cycles.
//
// The converse of `is_bipartite_no_odd_cycle` needs the missing bridge: an odd
// closed walk contains an odd cycle.  The proof takes an odd closed walk of
// minimal length; if its targets repeated a vertex, the walk would split into two
// shorter closed walks at that vertex, one of which is odd, contradicting
// minimality.  A minimal odd closed walk therefore has distinct targets, and with
// length at least three it is an odd cycle.  This section builds that argument.
// ============================================================================

/// A boolean inequality with the first side false forces the second side.
theorem bool_ne_not_left(p: Bool, q: Bool) {
    p != q and not p implies q
} by {
    if p != q and not p {
        if q {
            q
        }
        if not q {
            p = q
            p != q
            false
        }
        q
    }
}

/// A nonempty list has positive length.
theorem nonempty_length_pos[T](xs: List[T]) {
    xs != List.nil[T] implies Nat.0 < xs.length
} by {
    if xs != List.nil[T] {
        if xs.length = Nat.0 {
            length_zero_imp_nil(xs)
            xs = List.nil[T]
            false
        }
        xs.length != Nat.0
        pos_of_ne_zero(xs.length)
        Nat.0 < xs.length
    }
}

/// A closed walk has length different from one: a one-step closed walk would be a loop.
theorem not_closed_walk_length_one[V](g: SimpleGraph[V], x: V, steps: List[V]) {
    simple_graph_closed_walk(g, x, steps) implies not (steps.length = Nat.1)
} by {
    if simple_graph_closed_walk(g, x, steps) {
        if steps.length = Nat.1 {
            simple_graph_closed_walk_is_walk(g, x, steps)
            simple_graph_walk(g, x, x, steps)
            match steps {
                List.nil[V] {
                    steps.length = Nat.0
                    steps.length = Nat.1
                    Nat.0 = Nat.1
                    false
                }
                List.cons(h, t) {
                    steps.length = t.length + Nat.1
                    steps.length = Nat.1
                    t.length + Nat.1 = t.length.suc
                    t.length.suc = Nat.1
                    nat_suc_eq_cancel(t.length, Nat.0)
                    t.length = Nat.0
                    length_zero_imp_nil(t)
                    t = List.nil[V]
                    steps = List.cons(h, List.nil[V])
                    simple_graph_walk_cons_iff(g, x, x, h, List.nil[V])
                    simple_graph_walk(g, x, x, List.cons(h, List.nil[V])) =
                        (g.adj(x, h) and simple_graph_walk(g, h, x, List.nil[V]))
                    g.adj(x, h)
                    simple_graph_walk_nil(g, h, x)
                    simple_graph_walk(g, h, x, List.nil[V]) = (h = x)
                    h = x
                    g.adj(x, x)
                    simple_graph_adj_irreflexive(g, x)
                    not g.adj(x, x)
                    false
                }
            }
        }
        not (steps.length = Nat.1)
    }
}

/// A nonempty walk has its finish among its targets.
theorem walk_targets_contains_finish_of_nonempty[V](
    g: SimpleGraph[V], a: V, b: V, steps: List[V]
) {
    simple_graph_walk(g, a, b, steps) and steps.length > Nat.0 implies steps.contains(b)
} by {
    define p(xs: List[V]) -> Bool {
        forall(a2: V, b2: V) {
            simple_graph_walk(g, a2, b2, xs) and xs.length > Nat.0 implies xs.contains(b2)
        }
    }

    forall(a2: V, b2: V) {
        if simple_graph_walk(g, a2, b2, List.nil[V]) and List.nil[V].length > Nat.0 {
            List.nil[V].length = Nat.0
            Nat.0 > Nat.0
            not_lt_zero(Nat.0)
            false
        }
    }
    p(List.nil[V])

    forall(head: V, tail: List[V]) {
        if p(tail) {
            forall(a2: V, b2: V) {
                if simple_graph_walk(g, a2, b2, List.cons(head, tail)) and
                    List.cons(head, tail).length > Nat.0 {
                    simple_graph_walk_cons_iff(g, a2, b2, head, tail)
                    simple_graph_walk(g, a2, b2, List.cons(head, tail)) =
                        (g.adj(a2, head) and simple_graph_walk(g, head, b2, tail))
                    simple_graph_walk(g, head, b2, tail)
                    if tail = List.nil[V] {
                        simple_graph_walk_nil(g, head, b2)
                        simple_graph_walk(g, head, b2, List.nil[V]) = (head = b2)
                        head = b2
                        List.cons(head, List.nil[V]).contains(b2)
                        List.cons(head, tail).contains(b2)
                    }
                    if tail != List.nil[V] {
                        nonempty_length_pos(tail)
                        Nat.0 < tail.length
                        List.cons(head, tail).length = tail.length + Nat.1
                        p(tail)
                        (simple_graph_walk(g, head, b2, tail) and tail.length > Nat.0 implies
                            tail.contains(b2))
                        tail.length > Nat.0
                        tail.contains(b2)
                        cons_contains_of_tail_contains(head, tail, b2)
                        List.cons(head, tail).contains(b2)
                    }
                    List.cons(head, tail).contains(b2)
                }
                simple_graph_walk(g, a2, b2, List.cons(head, tail)) and
                    List.cons(head, tail).length > Nat.0 implies List.cons(head, tail).contains(b2)
            }
            p(List.cons(head, tail))
        }
    }
    forall(head: V, tail: List[V]) { p(tail) implies p(List.cons(head, tail)) }
    function[T0](x0: List[T0] -> Bool) {
        not forall(x1: T0, x2: List[T0]) { not x0(x2) or x0(List.cons(x1, x2)) } or
        not x0(List.nil[T0]) or forall(x3: List[T0]) { x0(x3) } = true
    }[V](p)
    function[T0](x0: List[T0] -> Bool, x1: List[T0]) {
        not forall(x2: T0, x3: List[T0]) { not x0(x3) or x0(List.cons(x2, x3)) } or
        not x0(List.nil[T0]) or x0(x1)
    }[V](p, steps)
    p(steps)
    p(steps) = forall(a2: V, b2: V) {
        simple_graph_walk(g, a2, b2, steps) and steps.length > Nat.0 implies steps.contains(b2)
    }
    if simple_graph_walk(g, a, b, steps) and steps.length > Nat.0 {
        simple_graph_walk(g, a, b, steps) and steps.length > Nat.0 implies steps.contains(b)
        steps.contains(b)
    }
}

/// A walk that stays inside `s` has every target in `s`.
theorem walk_in_set_contains_member[V](s: FiniteSet[V], steps: List[V], v: V) {
    walk_in_set(s, steps) and steps.contains(v) implies s.contains(v)
} by {
    define p(xs: List[V]) -> Bool {
        forall(v2: V) {
            walk_in_set(s, xs) and xs.contains(v2) implies s.contains(v2)
        }
    }

    forall(v2: V) {
        if walk_in_set(s, List.nil[V]) and List.nil[V].contains(v2) {
            not List.nil[V].contains(v2)
            false
        }
    }
    p(List.nil[V])

    forall(head: V, tail: List[V]) {
        if p(tail) {
            forall(v2: V) {
                if walk_in_set(s, List.cons(head, tail)) and List.cons(head, tail).contains(v2) {
                    walk_in_set_cons(s, head, tail)
                    walk_in_set(s, List.cons(head, tail)) = (s.contains(head) and walk_in_set(s, tail))
                    s.contains(head)
                    walk_in_set(s, tail)
                    cons_contains_eq(head, tail, v2)
                    List.cons(head, tail).contains(v2) = (v2 = head or tail.contains(v2))
                    if v2 = head {
                        s.contains(v2)
                    }
                    if tail.contains(v2) {
                        p(tail)
                        (walk_in_set(s, tail) and tail.contains(v2) implies s.contains(v2))
                        s.contains(v2)
                    }
                    s.contains(v2)
                }
            }
            p(List.cons(head, tail))
        }
    }
    forall(head: V, tail: List[V]) { p(tail) implies p(List.cons(head, tail)) }
    function[T0](x0: List[T0] -> Bool) {
        not forall(x1: T0, x2: List[T0]) { not x0(x2) or x0(List.cons(x1, x2)) } or
        not x0(List.nil[T0]) or forall(x3: List[T0]) { x0(x3) } = true
    }[V](p)
    function[T0](x0: List[T0] -> Bool, x1: List[T0]) {
        not forall(x2: T0, x3: List[T0]) { not x0(x3) or x0(List.cons(x2, x3)) } or
        not x0(List.nil[T0]) or x0(x1)
    }[V](p, steps)
    p(steps)
    p(steps) = forall(v2: V) {
        walk_in_set(s, steps) and steps.contains(v2) implies s.contains(v2)
    }
    if walk_in_set(s, steps) and steps.contains(v) {
        walk_in_set(s, steps) and steps.contains(v) implies s.contains(v)
        s.contains(v)
    }
}

/// A list that is not unique contains some item at two positions: at its first
/// occurrence and again later.
theorem not_unique_imp_two_occurrences[T](xs: List[T]) {
    not xs.is_unique implies exists(w: T) {
        xs.contains(w) and xs.drop(xs.find_first_idx(w) + Nat.1).contains(w)
    }
} by {
    define p(ys: List[T]) -> Bool {
        not ys.is_unique implies exists(w: T) {
            ys.contains(w) and ys.drop(ys.find_first_idx(w) + Nat.1).contains(w)
        }
    }

    if not List.nil[T].is_unique {
        List.nil[T].is_unique
        false
    }
    p(List.nil[T])

    forall(head: T, tail: List[T]) {
        if p(tail) {
            if not List.cons(head, tail).is_unique {
                if tail.contains(head) {
                    cons_contains_head(head, tail)
                    List.cons(head, tail).contains(head)
                    List.cons(head, tail).find_first_idx(head) = Nat.0
                    List.cons(head, tail).drop(Nat.0 + Nat.1) = List.cons(head, tail).drop(Nat.1)
                    drop_one(List.cons(head, tail))
                    List.cons(head, tail).drop(Nat.1) = List.cons(head, tail).tail
                    tail_cancels_cons(head, tail)
                    List.cons(head, tail).tail = tail
                    List.cons(head, tail).drop(Nat.0 + Nat.1) = tail
                    List.cons(head, tail).drop(List.cons(head, tail).find_first_idx(head) + Nat.1) = tail
                    List.cons(head, tail).drop(List.cons(head, tail).find_first_idx(head) + Nat.1).contains(head)
                    exists(w: T) {
                        List.cons(head, tail).contains(w) and
                            List.cons(head, tail).drop(List.cons(head, tail).find_first_idx(w) + Nat.1).contains(w)
                    }
                }
                if not tail.contains(head) {
                    if tail.is_unique {
                        cons_unique_of_tail_unique_not_contains(head, tail)
                        List.cons(head, tail).is_unique
                        false
                    }
                    not tail.is_unique
                    p(tail)
                    exists(w: T) {
                        tail.contains(w) and tail.drop(tail.find_first_idx(w) + Nat.1).contains(w)
                    }
                    let (w: T) satisfy {
                        tail.contains(w) and tail.drop(tail.find_first_idx(w) + Nat.1).contains(w)
                    }
                    tail.contains(w)
                    tail.drop(tail.find_first_idx(w) + Nat.1).contains(w)
                    cons_contains_of_tail_contains(head, tail, w)
                    List.cons(head, tail).contains(w)
                    if head = w {
                        tail.contains(head)
                        not tail.contains(head)
                        false
                    }
                    head != w
                    find_first_idx_cons(head, tail, w)
                    List.cons(head, tail).find_first_idx(w) = Nat.1 + tail.find_first_idx(w)
                    List.cons(head, tail).find_first_idx(w) + Nat.1 = Nat.1 + tail.find_first_idx(w) + Nat.1
                    drop_cons(head, tail, tail.find_first_idx(w) + Nat.1)
                    List.cons(head, tail).drop(tail.find_first_idx(w) + Nat.1 + Nat.1) =
                        tail.drop(tail.find_first_idx(w) + Nat.1)
                    List.cons(head, tail).drop(Nat.1 + tail.find_first_idx(w) + Nat.1) =
                        tail.drop(tail.find_first_idx(w) + Nat.1)
                    List.cons(head, tail).drop(List.cons(head, tail).find_first_idx(w) + Nat.1) =
                        tail.drop(tail.find_first_idx(w) + Nat.1)
                    List.cons(head, tail).drop(List.cons(head, tail).find_first_idx(w) + Nat.1).contains(w)
                    exists(u: T) {
                        List.cons(head, tail).contains(u) and
                            List.cons(head, tail).drop(List.cons(head, tail).find_first_idx(u) + Nat.1).contains(u)
                    }
                }
                exists(w: T) {
                    List.cons(head, tail).contains(w) and
                        List.cons(head, tail).drop(List.cons(head, tail).find_first_idx(w) + Nat.1).contains(w)
                }
            }
            p(List.cons(head, tail)) = (not List.cons(head, tail).is_unique implies exists(w: T) {
                List.cons(head, tail).contains(w) and
                    List.cons(head, tail).drop(List.cons(head, tail).find_first_idx(w) + Nat.1).contains(w)
            })
            p(List.cons(head, tail))
        }
    }
    forall(head: T, tail: List[T]) { p(tail) implies p(List.cons(head, tail)) }
    function[T0](x0: List[T0] -> Bool) {
        not forall(x1: T0, x2: List[T0]) { not x0(x2) or x0(List.cons(x1, x2)) } or
        not x0(List.nil[T0]) or forall(x3: List[T0]) { x0(x3) } = true
    }[T](p)
    function[T0](x0: List[T0] -> Bool, x1: List[T0]) {
        not forall(x2: T0, x3: List[T0]) { not x0(x3) or x0(List.cons(x2, x3)) } or
        not x0(List.nil[T0]) or x0(x1)
    }[T](p, xs)
    p(xs)
    p(xs) = (not xs.is_unique implies exists(w: T) {
        xs.contains(w) and xs.drop(xs.find_first_idx(w) + Nat.1).contains(w)
    })
    if not xs.is_unique {
        not xs.is_unique implies exists(w: T) {
            xs.contains(w) and xs.drop(xs.find_first_idx(w) + Nat.1).contains(w)
        }
        exists(w: T) {
            xs.contains(w) and xs.drop(xs.find_first_idx(w) + Nat.1).contains(w)
        }
    }
}

/// A walk splits canonically at the first occurrence of a target: the prefix is the
/// shortest walk to that target (nonempty, since the occurrence is a target), and the
/// suffix is exactly the list of targets after the first occurrence.
theorem walk_canonical_split[V](g: SimpleGraph[V], a: V, b: V, c: V, steps: List[V]) {
    simple_graph_walk(g, a, b, steps) and steps.contains(c) implies
        exists(pre: List[V], suf: List[V]) {
            steps = pre + suf and simple_graph_walk(g, a, c, pre) and simple_graph_walk(g, c, b, suf) and
            pre != List.nil[V] and suf = steps.drop(steps.find_first_idx(c) + Nat.1)
        }
} by {
    define pc(xs: List[V], a2: V, b2: V) -> Bool {
        simple_graph_walk(g, a2, b2, xs) and xs.contains(c) implies
            exists(pre: List[V], suf: List[V]) {
                xs = pre + suf and simple_graph_walk(g, a2, c, pre) and simple_graph_walk(g, c, b2, suf) and
                pre != List.nil[V] and suf = xs.drop(xs.find_first_idx(c) + Nat.1)
            }
    }

    define p(xs: List[V]) -> Bool {
        forall(a2: V, b2: V) {
            pc(xs, a2, b2)
        }
    }

    forall(a2: V, b2: V) {
        if simple_graph_walk(g, a2, b2, List.nil[V]) and List.nil[V].contains(c) {
            not List.nil[V].contains(c)
            false
        }
    }
    p(List.nil[V])

    forall(head: V, tail: List[V]) {
        if p(tail) {
            forall(a2: V, b2: V) {
                if simple_graph_walk(g, a2, b2, List.cons(head, tail)) and
                    List.cons(head, tail).contains(c) {
                    simple_graph_walk_cons_iff(g, a2, b2, head, tail)
                    simple_graph_walk(g, a2, b2, List.cons(head, tail)) =
                        (g.adj(a2, head) and simple_graph_walk(g, head, b2, tail))
                    simple_graph_walk(g, head, b2, tail)
                    if head = c {
                        List.cons(head, tail) = List.cons(head, List.nil[V]) + tail
                        simple_graph_walk_cons_iff(g, a2, c, head, List.nil[V])
                        simple_graph_walk(g, a2, c, List.cons(head, List.nil[V])) =
                            (g.adj(a2, head) and simple_graph_walk(g, head, c, List.nil[V]))
                        simple_graph_walk_nil(g, head, c)
                        simple_graph_walk(g, head, c, List.nil[V]) = (head = c)
                        simple_graph_walk(g, a2, c, List.cons(head, List.nil[V]))
                        List.cons(head, List.nil[V]) != List.nil[V]
                        List.cons(head, tail).find_first_idx(c) = Nat.0
                        List.cons(head, tail).drop(Nat.0 + Nat.1) = List.cons(head, tail).drop(Nat.1)
                        drop_one(List.cons(head, tail))
                        List.cons(head, tail).drop(Nat.1) = List.cons(head, tail).tail
                        tail_cancels_cons(head, tail)
                        List.cons(head, tail).tail = tail
                        List.cons(head, tail).drop(Nat.0 + Nat.1) = tail
                        List.cons(head, tail).drop(List.cons(head, tail).find_first_idx(c) + Nat.1) = tail
                        exists(pre: List[V], suf: List[V]) {
                            List.cons(head, tail) = pre + suf and
                                simple_graph_walk(g, a2, c, pre) and simple_graph_walk(g, c, b2, suf) and
                                pre != List.nil[V] and
                                suf = List.cons(head, tail).drop(List.cons(head, tail).find_first_idx(c) + Nat.1)
                        }
                    }
                    if head != c {
                        List.cons(head, tail).contains(c) = (c = head or tail.contains(c))
                        if c = head {
                            head = c
                            not head = c
                            false
                        }
                        tail.contains(c)
                        pc(tail, head, b2)
                        pc(tail, head, b2) = (simple_graph_walk(g, head, b2, tail) and tail.contains(c) implies
                            exists(pre: List[V], suf: List[V]) {
                                tail = pre + suf and simple_graph_walk(g, head, c, pre) and
                                    simple_graph_walk(g, c, b2, suf) and pre != List.nil[V] and
                                    suf = tail.drop(tail.find_first_idx(c) + Nat.1)
                            })
                        exists(pre: List[V], suf: List[V]) {
                            tail = pre + suf and simple_graph_walk(g, head, c, pre) and
                                simple_graph_walk(g, c, b2, suf) and pre != List.nil[V] and
                                suf = tail.drop(tail.find_first_idx(c) + Nat.1)
                        }
                        let (pre: List[V], suf: List[V]) satisfy {
                            tail = pre + suf and simple_graph_walk(g, head, c, pre) and
                                simple_graph_walk(g, c, b2, suf) and pre != List.nil[V] and
                                suf = tail.drop(tail.find_first_idx(c) + Nat.1)
                        }
                        List.cons(head, tail) = List.cons(head, pre) + suf
                        List.cons(head, pre) + suf = List.cons(head, pre + suf)
                        List.cons(head, tail) = List.cons(head, pre + suf)
                        tail = pre + suf
                        simple_graph_walk_cons_iff(g, a2, c, head, pre)
                        simple_graph_walk(g, a2, c, List.cons(head, pre)) =
                            (g.adj(a2, head) and simple_graph_walk(g, head, c, pre))
                        g.adj(a2, head)
                        simple_graph_walk(g, a2, c, List.cons(head, pre))
                        List.cons(head, pre) != List.nil[V]
                        find_first_idx_cons(head, tail, c)
                        List.cons(head, tail).find_first_idx(c) = Nat.1 + tail.find_first_idx(c)
                        List.cons(head, tail).find_first_idx(c) + Nat.1 = Nat.1 + tail.find_first_idx(c) + Nat.1
                        drop_cons(head, tail, tail.find_first_idx(c) + Nat.1)
                        List.cons(head, tail).drop(tail.find_first_idx(c) + Nat.1 + Nat.1) =
                            tail.drop(tail.find_first_idx(c) + Nat.1)
                        List.cons(head, tail).drop(Nat.1 + tail.find_first_idx(c) + Nat.1) =
                            tail.drop(tail.find_first_idx(c) + Nat.1)
                        List.cons(head, tail).drop(List.cons(head, tail).find_first_idx(c) + Nat.1) =
                            tail.drop(tail.find_first_idx(c) + Nat.1)
                        suf = tail.drop(tail.find_first_idx(c) + Nat.1)
                        List.cons(head, tail).drop(List.cons(head, tail).find_first_idx(c) + Nat.1) = suf
                        exists(p2: List[V], s2: List[V]) {
                            List.cons(head, tail) = p2 + s2 and
                                simple_graph_walk(g, a2, c, p2) and simple_graph_walk(g, c, b2, s2) and
                                p2 != List.nil[V] and
                                s2 = List.cons(head, tail).drop(List.cons(head, tail).find_first_idx(c) + Nat.1)
                        }
                    }
                    exists(pre: List[V], suf: List[V]) {
                        List.cons(head, tail) = pre + suf and
                            simple_graph_walk(g, a2, c, pre) and simple_graph_walk(g, c, b2, suf) and
                            pre != List.nil[V] and
                            suf = List.cons(head, tail).drop(List.cons(head, tail).find_first_idx(c) + Nat.1)
                    }
                }
                pc(List.cons(head, tail), a2, b2) = (simple_graph_walk(g, a2, b2, List.cons(head, tail)) and
                    List.cons(head, tail).contains(c) implies
                        exists(pre: List[V], suf: List[V]) {
                            List.cons(head, tail) = pre + suf and
                                simple_graph_walk(g, a2, c, pre) and simple_graph_walk(g, c, b2, suf) and
                                pre != List.nil[V] and
                                suf = List.cons(head, tail).drop(List.cons(head, tail).find_first_idx(c) + Nat.1)
                        })
                pc(List.cons(head, tail), a2, b2)
            }
            p(List.cons(head, tail)) = forall(a2: V, b2: V) {
                pc(List.cons(head, tail), a2, b2)
            }
            p(List.cons(head, tail))
        }
    }
    forall(head: V, tail: List[V]) { p(tail) implies p(List.cons(head, tail)) }
    function[T0](x0: List[T0] -> Bool) {
        not forall(x1: T0, x2: List[T0]) { not x0(x2) or x0(List.cons(x1, x2)) } or
        not x0(List.nil[T0]) or forall(x3: List[T0]) { x0(x3) } = true
    }[V](p)
    function[T0](x0: List[T0] -> Bool, x1: List[T0]) {
        not forall(x2: T0, x3: List[T0]) { not x0(x3) or x0(List.cons(x2, x3)) } or
        not x0(List.nil[T0]) or x0(x1)
    }[V](p, steps)
    p(steps)
    pc(steps, a, b)
    if simple_graph_walk(g, a, b, steps) and steps.contains(c) {
        pc(steps, a, b) = (simple_graph_walk(g, a, b, steps) and steps.contains(c) implies
            exists(pre: List[V], suf: List[V]) {
                steps = pre + suf and simple_graph_walk(g, a, c, pre) and simple_graph_walk(g, c, b, suf) and
                pre != List.nil[V] and suf = steps.drop(steps.find_first_idx(c) + Nat.1)
            })
        exists(pre: List[V], suf: List[V]) {
            steps = pre + suf and simple_graph_walk(g, a, c, pre) and simple_graph_walk(g, c, b, suf) and
            pre != List.nil[V] and suf = steps.drop(steps.find_first_idx(c) + Nat.1)
        }
    }
}

/// An odd closed walk contains an odd cycle.
///
/// Take an odd closed walk of minimal length.  If its targets repeated a vertex `w`,
/// the walk would split at the two occurrences of `w` into two shorter closed walks
/// at `w` whose lengths add up to the odd total, so one of them is odd — a shorter
/// odd closed walk, contradicting minimality.  A minimal odd closed walk therefore
/// has distinct targets; with length at least three (odd and not one), it is an odd
/// cycle.
theorem has_odd_closed_walk_imp_has_odd_cycle[V](g: SimpleGraph[V], s: FiniteSet[V]) {
    has_odd_closed_walk(g, s) implies has_odd_cycle(g, s)
} by {
    if has_odd_closed_walk(g, s) {
        has_odd_closed_walk(g, s) = exists(x: V, steps: List[V]) {
            simple_graph_closed_walk(g, x, steps) and nat_odd(steps.length) and walk_in_set(s, steps)
        }
        exists(x: V, steps: List[V]) {
            simple_graph_closed_walk(g, x, steps) and nat_odd(steps.length) and walk_in_set(s, steps)
        }
        let (x: V, steps: List[V]) satisfy {
            simple_graph_closed_walk(g, x, steps) and nat_odd(steps.length) and walk_in_set(s, steps)
        }
        define odd_closed_walk_of_length(n: Nat) -> Bool {
            exists(a: V, ws: List[V]) {
                simple_graph_closed_walk(g, a, ws) and nat_odd(ws.length) and walk_in_set(s, ws) and
                    ws.length = n
            }
        }
        odd_closed_walk_of_length(steps.length) = exists(a: V, ws: List[V]) {
            simple_graph_closed_walk(g, a, ws) and nat_odd(ws.length) and walk_in_set(s, ws) and
                ws.length = steps.length
        }
        exists(a: V, ws: List[V]) {
            simple_graph_closed_walk(g, a, ws) and nat_odd(ws.length) and walk_in_set(s, ws) and
                ws.length = steps.length
        }
        odd_closed_walk_of_length(steps.length)
        has_min(odd_closed_walk_of_length, steps.length)
        exists(m: Nat) { is_min(odd_closed_walk_of_length, m) }
        let (m: Nat) satisfy { is_min(odd_closed_walk_of_length, m) }
        is_min_apply(odd_closed_walk_of_length, m)
        odd_closed_walk_of_length(m)
        odd_closed_walk_of_length(m) = exists(a: V, ws: List[V]) {
            simple_graph_closed_walk(g, a, ws) and nat_odd(ws.length) and walk_in_set(s, ws) and
                ws.length = m
        }
        exists(a: V, ws: List[V]) {
            simple_graph_closed_walk(g, a, ws) and nat_odd(ws.length) and walk_in_set(s, ws) and
                ws.length = m
        }
        let (y: V, ws: List[V]) satisfy {
            simple_graph_closed_walk(g, y, ws) and nat_odd(ws.length) and walk_in_set(s, ws) and
                ws.length = m
        }
        simple_graph_closed_walk(g, y, ws)
        nat_odd(ws.length)
        walk_in_set(s, ws)
        ws.length = m
        if ws.is_unique {
            not_closed_walk_length_one(g, y, ws)
            not (ws.length = Nat.1)
            if ws.length = Nat.1 {
                ws.length = Nat.1
                not (ws.length = Nat.1)
                false
            }
            ws.length != Nat.1
            nat_odd_ne_zero(ws.length)
            ws.length != Nat.0
            if ws.length = Nat.0 {
                ws.length = Nat.0
                ws.length != Nat.0
                false
            }
            ws.length != Nat.0
            if ws.length = Nat.2 {
                nat_odd(ws.length) = nat_odd(Nat.2)
                nat_odd(Nat.2) = false
                nat_odd(ws.length) = false
                nat_odd(ws.length)
                false
            }
            ws.length != Nat.2
            if ws.length < Nat.3 {
                nat_lt_three_cases(ws.length)
                ws.length = Nat.0 or ws.length = Nat.1 or ws.length = Nat.2
                if ws.length = Nat.0 {
                    ws.length != Nat.0
                    false
                }
                if ws.length = Nat.1 {
                    ws.length != Nat.1
                    false
                }
                if ws.length = Nat.2 {
                    ws.length != Nat.2
                    false
                }
                false
            }
            not (ws.length < Nat.3)
            lt_or_lte(ws.length, Nat.3)
            ws.length < Nat.3 or Nat.3 <= ws.length
            Nat.3 <= ws.length
            Nat.0 < Nat.3
            lt_and_lte(Nat.0, Nat.3, ws.length)
            Nat.0 < ws.length
            ws.length > Nat.0
            simple_graph_closed_walk_is_walk(g, y, ws)
            simple_graph_walk(g, y, y, ws)
            walk_targets_contains_finish_of_nonempty(g, y, y, ws)
            ws.contains(y)
            walk_in_set_contains_member(s, ws, y)
            s.contains(y)
            simple_graph_cycle(g, y, ws) = (simple_graph_closed_walk(g, y, ws) and
                Nat.3 <= ws.length and ws.is_unique)
            simple_graph_cycle(g, y, ws)
            has_odd_cycle(g, s) = exists(start: V, csteps: List[V]) {
                s.contains(start) and simple_graph_cycle(g, start, csteps) and
                    nat_odd(csteps.length) and walk_in_set(s, csteps)
            }
            exists(start: V, csteps: List[V]) {
                s.contains(start) and simple_graph_cycle(g, start, csteps) and
                    nat_odd(csteps.length) and walk_in_set(s, csteps)
            }
            has_odd_cycle(g, s)
        }
        if not ws.is_unique {
            not_unique_imp_two_occurrences(ws)
            exists(w: V) {
                ws.contains(w) and ws.drop(ws.find_first_idx(w) + Nat.1).contains(w)
            }
            let (w: V) satisfy {
                ws.contains(w) and ws.drop(ws.find_first_idx(w) + Nat.1).contains(w)
            }
            ws.contains(w)
            ws.drop(ws.find_first_idx(w) + Nat.1).contains(w)
            simple_graph_closed_walk_is_walk(g, y, ws)
            simple_graph_walk(g, y, y, ws)
            walk_canonical_split(g, y, y, w, ws)
            exists(pre: List[V], suf: List[V]) {
                ws = pre + suf and simple_graph_walk(g, y, w, pre) and
                    simple_graph_walk(g, w, y, suf) and pre != List.nil[V] and
                    suf = ws.drop(ws.find_first_idx(w) + Nat.1)
            }
            let (pre: List[V], suf: List[V]) satisfy {
                ws = pre + suf and simple_graph_walk(g, y, w, pre) and
                    simple_graph_walk(g, w, y, suf) and pre != List.nil[V] and
                    suf = ws.drop(ws.find_first_idx(w) + Nat.1)
            }
            ws = pre + suf
            simple_graph_walk(g, y, w, pre)
            simple_graph_walk(g, w, y, suf)
            pre != List.nil[V]
            suf = ws.drop(ws.find_first_idx(w) + Nat.1)
            suf.contains(w)
            walk_canonical_split(g, w, y, w, suf)
            exists(pre2: List[V], suf2: List[V]) {
                suf = pre2 + suf2 and simple_graph_walk(g, w, w, pre2) and
                    simple_graph_walk(g, w, y, suf2) and pre2 != List.nil[V] and
                    suf2 = suf.drop(suf.find_first_idx(w) + Nat.1)
            }
            let (pre2: List[V], suf2: List[V]) satisfy {
                suf = pre2 + suf2 and simple_graph_walk(g, w, w, pre2) and
                    simple_graph_walk(g, w, y, suf2) and pre2 != List.nil[V] and
                    suf2 = suf.drop(suf.find_first_idx(w) + Nat.1)
            }
            suf = pre2 + suf2
            simple_graph_walk(g, w, w, pre2)
            simple_graph_walk(g, w, y, suf2)
            pre2 != List.nil[V]
            nonempty_length_pos(pre2)
            Nat.0 < pre2.length
            nonempty_length_pos(pre)
            Nat.0 < pre.length
            add_length(pre, suf)
            (pre + suf).length = pre.length + suf.length
            ws.length = pre.length + suf.length
            add_length(pre2, suf2)
            (pre2 + suf2).length = pre2.length + suf2.length
            suf.length = pre2.length + suf2.length
            ws.length = pre.length + (pre2.length + suf2.length)
            add_assoc(pre.length, pre2.length, suf2.length)
            (pre.length + pre2.length) + suf2.length = pre.length + (pre2.length + suf2.length)
            add_comm(pre.length, pre2.length)
            pre.length + pre2.length = pre2.length + pre.length
            add_assoc(pre2.length, pre.length, suf2.length)
            (pre2.length + pre.length) + suf2.length = pre2.length + (pre.length + suf2.length)
            add_comm(pre.length, suf2.length)
            pre.length + suf2.length = suf2.length + pre.length
            ws.length = pre2.length + (suf2.length + pre.length)
            nat_odd_add(pre2.length, suf2.length + pre.length)
            nat_odd(pre2.length + (suf2.length + pre.length)) =
                (nat_odd(pre2.length) != nat_odd(suf2.length + pre.length))
            nat_odd(ws.length) = (nat_odd(pre2.length) != nat_odd(suf2.length + pre.length))
            nat_odd(ws.length)
            nat_odd(pre2.length) != nat_odd(suf2.length + pre.length)
            lt_add_left(suf2.length, Nat.0, pre.length)
            Nat.0 < pre.length implies suf2.length + Nat.0 < suf2.length + pre.length
            suf2.length + Nat.0 < suf2.length + pre.length
            suf2.length < suf2.length + pre.length
            Nat.0 <= suf2.length
            lte_and_lt(Nat.0, suf2.length, suf2.length + pre.length)
            Nat.0 < suf2.length + pre.length
            if nat_odd(pre2.length) {
                simple_graph_closed_walk_intro(g, w, pre2)
                simple_graph_closed_walk(g, w, pre2)
                walk_in_set(s, ws)
                ws = pre + suf
                walk_in_set(s, pre + suf)
                walk_in_set_left(s, pre, suf)
                walk_in_set(s, pre)
                walk_in_set_right(s, pre, suf)
                walk_in_set(s, suf)
                suf = pre2 + suf2
                walk_in_set(s, pre2 + suf2)
                walk_in_set_left(s, pre2, suf2)
                walk_in_set(s, pre2)
                odd_closed_walk_of_length(pre2.length) = exists(a: V, ws2: List[V]) {
                    simple_graph_closed_walk(g, a, ws2) and nat_odd(ws2.length) and
                        walk_in_set(s, ws2) and ws2.length = pre2.length
                }
                exists(a: V, ws2: List[V]) {
                    simple_graph_closed_walk(g, a, ws2) and nat_odd(ws2.length) and
                        walk_in_set(s, ws2) and ws2.length = pre2.length
                }
                odd_closed_walk_of_length(pre2.length)
                lt_add_left(pre2.length, Nat.0, suf2.length + pre.length)
                (Nat.0 < suf2.length + pre.length implies
                    pre2.length + Nat.0 < pre2.length + (suf2.length + pre.length))
                pre2.length + Nat.0 < pre2.length + (suf2.length + pre.length)
                pre2.length < ws.length
                ws.length = m
                pre2.length < m
                is_min_false_below(odd_closed_walk_of_length, m)
                false_below(odd_closed_walk_of_length, m)
                false_below_apply(odd_closed_walk_of_length, m, pre2.length)
                not odd_closed_walk_of_length(pre2.length)
                false
            }
            if not nat_odd(pre2.length) {
                bool_ne_not_left(nat_odd(pre2.length), nat_odd(suf2.length + pre.length))
                nat_odd(suf2.length + pre.length)
                simple_graph_walk_concat(g, w, y, w, suf2, pre)
                simple_graph_walk(g, w, w, suf2 + pre)
                add_length(suf2, pre)
                (suf2 + pre).length = suf2.length + pre.length
                Nat.0 < suf2.length + pre.length
                (suf2 + pre).length > Nat.0
                simple_graph_closed_walk_intro(g, w, suf2 + pre)
                simple_graph_closed_walk(g, w, suf2 + pre)
                nat_odd((suf2 + pre).length) = nat_odd(suf2.length + pre.length)
                nat_odd((suf2 + pre).length)
                walk_in_set(s, ws)
                ws = pre + suf
                walk_in_set(s, pre + suf)
                walk_in_set_left(s, pre, suf)
                walk_in_set(s, pre)
                walk_in_set_right(s, pre, suf)
                walk_in_set(s, suf)
                suf = pre2 + suf2
                walk_in_set(s, pre2 + suf2)
                walk_in_set_right(s, pre2, suf2)
                walk_in_set(s, suf2)
                walk_in_set_concat(s, suf2, pre)
                walk_in_set(s, suf2 + pre)
                odd_closed_walk_of_length(suf2.length + pre.length) = exists(a: V, ws2: List[V]) {
                    simple_graph_closed_walk(g, a, ws2) and nat_odd(ws2.length) and
                        walk_in_set(s, ws2) and ws2.length = suf2.length + pre.length
                }
                exists(a: V, ws2: List[V]) {
                    simple_graph_closed_walk(g, a, ws2) and nat_odd(ws2.length) and
                        walk_in_set(s, ws2) and ws2.length = suf2.length + pre.length
                }
                odd_closed_walk_of_length(suf2.length + pre.length)
                add_comm(pre2.length, suf2.length + pre.length)
                pre2.length + (suf2.length + pre.length) = (suf2.length + pre.length) + pre2.length
                ws.length = (suf2.length + pre.length) + pre2.length
                lt_add_left(suf2.length + pre.length, Nat.0, pre2.length)
                (Nat.0 < pre2.length implies
                    (suf2.length + pre.length) + Nat.0 < (suf2.length + pre.length) + pre2.length)
                (suf2.length + pre.length) + Nat.0 < (suf2.length + pre.length) + pre2.length
                suf2.length + pre.length < ws.length
                ws.length = m
                suf2.length + pre.length < m
                is_min_false_below(odd_closed_walk_of_length, m)
                false_below(odd_closed_walk_of_length, m)
                false_below_apply(odd_closed_walk_of_length, m, suf2.length + pre.length)
                not odd_closed_walk_of_length(suf2.length + pre.length)
                false
            }
            false
        }
        has_odd_cycle(g, s)
    }
}

// ============================================================================
// The odd-cycle characterisation of two-colourability, converse direction.
//
// With the bridge above, a graph with no odd cycle has no odd closed walk, so the
// parity colouring of `simple_graph_bipartite_odd_cycle` is a bipartition, and the
// graph is two-colourable.  The connectedness hypothesis is the one used by that
// colouring: it colours by the parity of a walk from a root, which is well defined
// on each connected component.
// ============================================================================

/// A graph with no odd cycle is two-colourable (on a vertex set connected from the root).
theorem is_k_colorable_two_of_no_odd_cycle[V](
    g: SimpleGraph[V], s: FiniteSet[V], r: V
) {
    s.contains(r) and walk_reachable_from(g, s, r) and not has_odd_cycle(g, s)
        implies is_k_colorable(g, s, Nat.2)
} by {
    if s.contains(r) and walk_reachable_from(g, s, r) and not has_odd_cycle(g, s) {
        if has_odd_closed_walk(g, s) {
            has_odd_closed_walk_imp_has_odd_cycle(g, s)
            has_odd_cycle(g, s)
            not has_odd_cycle(g, s)
            false
        }
        not has_odd_closed_walk(g, s)
        is_bipartite_of_no_odd_closed_walk(g, s, r)
        is_bipartite(g, s)
        is_bipartite_imp_is_k_colorable_two(g, s)
        is_k_colorable(g, s, Nat.2)
    }
}

/// A graph is two-colourable exactly when it has no odd cycle (on a vertex set
/// connected from the root `r`).
theorem is_k_colorable_two_eq_no_odd_cycle[V](
    g: SimpleGraph[V], s: FiniteSet[V], r: V
) {
    s.contains(r) and walk_reachable_from(g, s, r) implies
        (is_k_colorable(g, s, Nat.2) = not has_odd_cycle(g, s))
} by {
    if s.contains(r) and walk_reachable_from(g, s, r) {
        is_k_colorable_two_imp_no_odd_cycle(g, s)
        is_k_colorable(g, s, Nat.2) implies not has_odd_cycle(g, s)
        is_k_colorable_two_of_no_odd_cycle(g, s, r)
        not has_odd_cycle(g, s) implies is_k_colorable(g, s, Nat.2)
        is_k_colorable(g, s, Nat.2) = not has_odd_cycle(g, s)
    }
}

// ============================================================================
// Trees are two-colourable.
//
// A tree is acyclic, hence has no odd cycle, hence (by the section above) has no
// odd closed walk.  A tree is also connected, so the parity colouring from any
// root is a bipartition, and the tree is two-colourable.  The bridge lemmas below
// translate the library's connectedness predicate (reachability in the induced
// subgraph) into the explicit-walk reachability used by the parity colouring.
// ============================================================================

/// An explicit walk of the induced subgraph is an explicit walk of the base graph whose
/// targets all lie in `s`.
theorem induced_walk_imp_base_walk_in_set[V](
    g: SimpleGraph[V], s: FiniteSet[V], a: V, b: V, steps: List[V]
) {
    simple_graph_walk(induced_subgraph(g, s.underlying_set), a, b, steps) implies
        simple_graph_walk(g, a, b, steps) and walk_in_set(s, steps)
} by {
    define pc(xs: List[V], a2: V, b2: V) -> Bool {
        simple_graph_walk(induced_subgraph(g, s.underlying_set), a2, b2, xs) implies
            simple_graph_walk(g, a2, b2, xs) and walk_in_set(s, xs)
    }

    define p(xs: List[V]) -> Bool {
        forall(a2: V, b2: V) {
            pc(xs, a2, b2)
        }
    }

    forall(a2: V, b2: V) {
        pc(List.nil[V], a2, b2) = (
            simple_graph_walk(induced_subgraph(g, s.underlying_set), a2, b2, List.nil[V]) implies
                simple_graph_walk(g, a2, b2, List.nil[V]) and walk_in_set(s, List.nil[V]))
        if simple_graph_walk(induced_subgraph(g, s.underlying_set), a2, b2, List.nil[V]) {
            simple_graph_walk_nil(induced_subgraph(g, s.underlying_set), a2, b2)
            a2 = b2
            simple_graph_walk_nil(g, a2, b2)
            simple_graph_walk(g, a2, b2, List.nil[V])
            walk_in_set_nil(s)
            walk_in_set(s, List.nil[V])
            simple_graph_walk(g, a2, b2, List.nil[V]) and walk_in_set(s, List.nil[V])
        }
        pc(List.nil[V], a2, b2)
    }
    p(List.nil[V])

    forall(head: V, tail: List[V]) {
        if p(tail) {
            forall(a2: V, b2: V) {
                if simple_graph_walk(induced_subgraph(g, s.underlying_set), a2, b2,
                    List.cons(head, tail)) {
                    simple_graph_walk_cons_iff(induced_subgraph(g, s.underlying_set), a2, b2,
                        head, tail)
                    induced_subgraph(g, s.underlying_set).adj(a2, head)
                    simple_graph_walk(induced_subgraph(g, s.underlying_set), head, b2, tail)
                    induced_subgraph_adj_eq(g, s.underlying_set, a2, head)
                    induced_subgraph(g, s.underlying_set).adj(a2, head) =
                        (g.adj(a2, head) and s.contains(head))
                    g.adj(a2, head)
                    s.contains(head)
                    pc(tail, head, b2)
                    pc(tail, head, b2) = (
                        simple_graph_walk(induced_subgraph(g, s.underlying_set), head, b2, tail)
                            implies simple_graph_walk(g, head, b2, tail) and walk_in_set(s, tail))
                    simple_graph_walk(g, head, b2, tail) and walk_in_set(s, tail)
                    simple_graph_walk(g, head, b2, tail)
                    walk_in_set(s, tail)
                    simple_graph_walk_cons_iff(g, a2, b2, head, tail)
                    simple_graph_walk(g, a2, b2, List.cons(head, tail)) =
                        (g.adj(a2, head) and simple_graph_walk(g, head, b2, tail))
                    simple_graph_walk(g, a2, b2, List.cons(head, tail))
                    walk_in_set_cons(s, head, tail)
                    walk_in_set(s, List.cons(head, tail)) = (s.contains(head) and walk_in_set(s, tail))
                    walk_in_set(s, List.cons(head, tail))
                    simple_graph_walk(g, a2, b2, List.cons(head, tail)) and
                        walk_in_set(s, List.cons(head, tail))
                }
                pc(List.cons(head, tail), a2, b2) = (
                    simple_graph_walk(induced_subgraph(g, s.underlying_set), a2, b2,
                        List.cons(head, tail)) implies
                            simple_graph_walk(g, a2, b2, List.cons(head, tail)) and
                                walk_in_set(s, List.cons(head, tail)))
                pc(List.cons(head, tail), a2, b2)
            }
            p(List.cons(head, tail)) = forall(a2: V, b2: V) {
                pc(List.cons(head, tail), a2, b2)
            }
            p(List.cons(head, tail))
        }
    }
    forall(head: V, tail: List[V]) { p(tail) implies p(List.cons(head, tail)) }
    function[T0](x0: List[T0] -> Bool) {
        not forall(x1: T0, x2: List[T0]) { not x0(x2) or x0(List.cons(x1, x2)) } or
        not x0(List.nil[T0]) or forall(x3: List[T0]) { x0(x3) } = true
    }[V](p)
    function[T0](x0: List[T0] -> Bool, x1: List[T0]) {
        not forall(x2: T0, x3: List[T0]) { not x0(x3) or x0(List.cons(x2, x3)) } or
        not x0(List.nil[T0]) or x0(x1)
    }[V](p, steps)
    p(steps)
    p(steps) = forall(a2: V, b2: V) {
        pc(steps, a2, b2)
    }
    forall(a2: V, b2: V) {
        pc(steps, a2, b2)
    }
    pc(steps, a, b)
    pc(steps, a, b) = (
        simple_graph_walk(induced_subgraph(g, s.underlying_set), a, b, steps) implies
            simple_graph_walk(g, a, b, steps) and walk_in_set(s, steps))
    if simple_graph_walk(induced_subgraph(g, s.underlying_set), a, b, steps) {
        simple_graph_walk(g, a, b, steps) and walk_in_set(s, steps)
    }
}

/// Connectedness on `s` gives explicit walks from any root of `s` to every vertex of `s`
/// that stay inside `s`.
theorem connected_on_imp_walk_reachable_helper[V](g: SimpleGraph[V], s: FiniteSet[V], r: V) {
    simple_graph_connected_on(g, s) and s.contains(r) implies
        forall(x: V) {
            s.contains(x) implies exists(steps: List[V]) {
                simple_graph_walk(g, r, x, steps) and walk_in_set(s, steps)
            }
        }
} by {
    if simple_graph_connected_on(g, s) and s.contains(r) {
        forall(x: V) {
            if s.contains(x) {
                simple_graph_connected_on(g, s) = forall(a: V, b: V) {
                    s.contains(a) and s.contains(b) implies
                        simple_graph_reachable(induced_subgraph(g, s.underlying_set), a, b)
                }
                forall(a: V, b: V) {
                    (s.contains(a) and s.contains(b) implies
                        simple_graph_reachable(induced_subgraph(g, s.underlying_set), a, b))
                }
                (s.contains(r) and s.contains(x) implies
                    simple_graph_reachable(induced_subgraph(g, s.underlying_set), r, x))
                simple_graph_reachable(induced_subgraph(g, s.underlying_set), r, x)
                simple_graph_reachable_has_walk(induced_subgraph(g, s.underlying_set), r, x)
                exists(steps: List[V]) {
                    simple_graph_walk(induced_subgraph(g, s.underlying_set), r, x, steps)
                }
                let (steps: List[V]) satisfy {
                    simple_graph_walk(induced_subgraph(g, s.underlying_set), r, x, steps)
                }
                induced_walk_imp_base_walk_in_set(g, s, r, x, steps)
                simple_graph_walk(g, r, x, steps) and walk_in_set(s, steps)
                exists(ws: List[V]) {
                    simple_graph_walk(g, r, x, ws) and walk_in_set(s, ws)
                }
            }
        }
    }
}

/// Connectedness on `s` gives `walk_reachable_from` from any root of `s`.
theorem connected_on_imp_walk_reachable_from[V](g: SimpleGraph[V], s: FiniteSet[V], r: V) {
    simple_graph_connected_on(g, s) and s.contains(r) implies walk_reachable_from(g, s, r)
} by {
    if simple_graph_connected_on(g, s) and s.contains(r) {
        connected_on_imp_walk_reachable_helper(g, s, r)
        forall(x: V) {
            s.contains(x) implies exists(steps: List[V]) {
                simple_graph_walk(g, r, x, steps) and walk_in_set(s, steps)
            }
        }
        walk_reachable_from(g, s, r) = forall(x: V) {
            s.contains(x) implies exists(steps: List[V]) {
                simple_graph_walk(g, r, x, steps) and walk_in_set(s, steps)
            }
        }
        walk_reachable_from(g, s, r)
    }
}

/// An acyclic graph has no odd cycle.
theorem acyclic_imp_no_odd_cycle[V](g: SimpleGraph[V], s: FiniteSet[V]) {
    is_acyclic(g, s) implies not has_odd_cycle(g, s)
} by {
    if is_acyclic(g, s) {
        if has_odd_cycle(g, s) {
            has_odd_cycle(g, s) = exists(start: V, steps: List[V]) {
                s.contains(start) and simple_graph_cycle(g, start, steps) and
                    nat_odd(steps.length) and walk_in_set(s, steps)
            }
            exists(start: V, steps: List[V]) {
                s.contains(start) and simple_graph_cycle(g, start, steps) and
                    nat_odd(steps.length) and walk_in_set(s, steps)
            }
            let (start: V, steps: List[V]) satisfy {
                s.contains(start) and simple_graph_cycle(g, start, steps) and
                    nat_odd(steps.length) and walk_in_set(s, steps)
            }
            s.contains(start)
            simple_graph_cycle(g, start, steps)
            walk_in_set(s, steps)
            forall(v: V) {
                if steps.contains(v) {
                    walk_in_set_contains_member(s, steps, v)
                    s.contains(v)
                }
                steps.contains(v) implies s.contains(v)
            }
            is_acyclic_cycle_contradiction(g, s, start, steps)
            not (is_acyclic(g, s) and s.contains(start) and simple_graph_cycle(g, start, steps) and
                (forall(x: V) { steps.contains(x) implies s.contains(x) }))
            is_acyclic(g, s) and s.contains(start) and simple_graph_cycle(g, start, steps) and
                (forall(x: V) { steps.contains(x) implies s.contains(x) })
            false
        }
        not has_odd_cycle(g, s)
    }
}

/// A graph on an empty vertex set is two-colourable: the colouring condition is vacuous.
theorem empty_set_two_colorable[V](g: SimpleGraph[V], s: FiniteSet[V]) {
    (forall(r: V) { not s.contains(r) }) implies is_k_colorable(g, s, Nat.2)
} by {
    if forall(r: V) { not s.contains(r) } {
        is_k_colorable_intro(g, s, Nat.2, nat_const_fn[V](Nat.0))
        is_proper_coloring_on_intro(g, s, nat_const_fn[V](Nat.0))
        forall(x: V, y: V) {
            if s.contains(x) and s.contains(y) and g.adj(x, y) {
                not s.contains(x)
                false
            }
        }
        forall(v: V) {
            if s.contains(v) {
                not s.contains(v)
                false
            }
        }
        is_proper_coloring_on(g, s, nat_const_fn[V](Nat.0))
        is_k_colorable_intro(g, s, Nat.2, nat_const_fn[V](Nat.0))
        is_k_colorable(g, s, Nat.2)
    }
}

/// Every tree is two-colourable.
///
/// A tree is connected and acyclic; acyclicity rules out odd closed walks (through the
/// odd-cycle bridge), so the parity colouring from any root is a bipartition.
theorem tree_is_k_colorable_two[V](g: SimpleGraph[V], s: FiniteSet[V]) {
    is_tree(g, s) implies is_k_colorable(g, s, Nat.2)
} by {
    if is_tree(g, s) {
        tree_acyclic(g, s)
        is_acyclic(g, s)
        acyclic_imp_no_odd_cycle(g, s)
        not has_odd_cycle(g, s)
        tree_connected_on(g, s)
        simple_graph_connected_on(g, s)
        if exists(r: V) { s.contains(r) } {
            let (r: V) satisfy { s.contains(r) }
            s.contains(r)
            connected_on_imp_walk_reachable_from(g, s, r)
            walk_reachable_from(g, s, r)
            is_k_colorable_two_of_no_odd_cycle(g, s, r)
            is_k_colorable(g, s, Nat.2)
        }
        if not exists(r: V) { s.contains(r) } {
            not_exists_forward(function(r: V) { s.contains(r) })
            forall(r: V) { not s.contains(r) }
            empty_set_two_colorable(g, s)
            is_k_colorable(g, s, Nat.2)
        }
        is_k_colorable(g, s, Nat.2)
    }
}

// ============================================================================
// The greedy bound (statement).
//
// If every vertex of `s` has at most `delta` neighbours, then `delta + 1` colours
// suffice: colour the vertices of `s` one at a time, and when a vertex is reached
// at most `delta` colours are already used on its neighbours, so at least one of
// the `delta + 1` colours is free.  Formalising this requires building the greedy
// colouring by induction over an enumeration of the finite vertex set, with a
// "least free colour" step at each vertex; that construction is left for future
// work, so the theorem is recorded here as a statement only (compare the note in
// `simple_graph_chromatic.ac`).
//
// theorem greedy_coloring_bound[V](g: SimpleGraph[V], s: FiniteSet[V], delta: Nat) {
//     max_degree_at_most(g, s, delta) implies is_k_colorable(g, s, delta + Nat.1)
// }

// ============================================================================
// The five-colour theorem (statement).
//
// Every planar graph is five-colourable.  The proof needs the fact that every
// planar graph on a nonempty vertex set has a vertex of degree at most five: by
// the planar edge bound `E <= 3V - 6` (`planar_edge_bound`) and the degree sum
// `2E = sum_v deg(v)`, if every vertex had degree at least six then `E >= 3V`,
// contradicting the bound.  Removing such a vertex, colouring the rest by
// induction, and giving the removed vertex a colour not used by its (at most five)
// neighbours proves the theorem.  The planarity predicate of this library is still
// a placeholder (`is_planar` is constant `false`), so the degree lemma and the
// theorem itself are recorded below as statements only; they become obligations
// when a genuine embedding definition replaces the placeholder.
// ============================================================================

// /// Every planar graph on a nonempty vertex set has a vertex of degree at most five.
// ///
// /// The classical proof doubles the face-edge incidences to get the planar edge bound
// /// `E <= 3V - 6` (`planar_edge_bound`) and compares the degree sum `2E = sum_v deg(v)`:
// /// if every vertex had degree at least six, then `E >= 3V`, contradicting the bound.
// /// The statement is vacuous under the placeholder `is_planar` and is recorded here for
// /// the five-colour proof below.
// theorem planar_has_vertex_degree_le_five[V](g: SimpleGraph[V], s: FiniteSet[V]) {
//     is_planar(g) and Nat.0 < fs_card(s) implies exists(v: V) {
//         s.contains(v) and degree(g, s, v) <= Nat.5
//     }
// }

// /// The five-colour theorem: every planar graph is five-colourable.
// ///
// /// The proof goes by induction on the number of vertices: a planar graph has a
// /// vertex of degree at most five (`planar_has_vertex_degree_le_five`); removing it
// /// leaves a planar graph, which is five-colourable by induction, and at most five
// /// colours are used on its neighbours, so one of the five colours is free for it.
// /// Like the four-colour theorem, the formal statement awaits a genuine notion of
// /// planarity; the current placeholder `is_planar` would make it vacuous.
// theorem five_color_theorem[V](g: SimpleGraph[V], s: FiniteSet[V]) {
//     is_planar(g) implies is_k_colorable(g, s, Nat.5)
// }
