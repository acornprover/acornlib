from nat import Nat, lte_trans, lt_and_lte, small_mod
from pair import Pair
from finite_set import FiniteSet
from data.finite.finite_set_card import fs_card
from data.finite.finite_set_product import finite_set_product, finite_set_product_contains_pair
from data.finite.finite_set_product_card import fs_card_product
from data.finite.finite_set_cover_card import is_covered_by_image, is_covered_by_image_intro,
    fs_card_le_of_covered_by_image, has_preimage_in, has_preimage_in_intro
from data.nat.nat_mod_period import add_mod_self, small_add_mod_self, suc_mod_cases
from data.nat.nat_range_set import range_set, range_set_contains, range_set_lt, range_set_card
from graph.simple_graph_domination import is_dominated, is_dominating_set, is_dominating_set_apply,
    has_neighbor_in, has_neighbor_in_witness
from graph.simple_graph_domination_number import domination_number, domination_number_attained,
    dominating_size_pred
from graph.simple_graph_cycle import cycle_graph, cycle_graph_adj_iff
from graph.simple_graph_path_domination import three_offsets_lt

numerals Nat

/// The vertex of the cycle reached from a class and an offset.
///
/// A vertex dominated by `w` is one of `w - 1`, `w`, and `w + 1` around the cycle, which are
/// `w + n + 0 - 1`, `w + n + 1 - 1`, and `w + n + 2 - 1` reduced modulo `n`. Adding a whole
/// turn before subtracting one keeps every intermediate value a natural, so the truncated
/// subtraction is exact at every offset including the wrap.
define cycle_offset_vertex(n: Nat, p: Pair[Nat, Nat]) -> Nat {
    (p.first + n + p.second - Nat.1).mod(n)
}

/// The offset that recovers a vertex from itself.
theorem cycle_offset_self(n: Nat, w: Nat) {
    w < n implies cycle_offset_vertex(n, Pair.new(w, Nat.1)) = w
} by {
    if w < n {
        (Pair.new(w, Nat.1).first = w)
        (Pair.new(w, Nat.1).second = Nat.1)
        (cycle_offset_vertex(n, Pair.new(w, Nat.1)) = (w + n + Nat.1 - Nat.1).mod(n))
        (w + n + Nat.1 - Nat.1 = w + n)
        small_add_mod_self(w, n)
        (w + n).mod(n) = w
        cycle_offset_vertex(n, Pair.new(w, Nat.1)) = w
    }
}

/// The offset that recovers the successor of a vertex.
theorem cycle_offset_suc(n: Nat, w: Nat) {
    n != Nat.0 implies cycle_offset_vertex(n, Pair.new(w, Nat.2)) = w.suc.mod(n)
} by {
    if n != Nat.0 {
        (Pair.new(w, Nat.2).first = w)
        (Pair.new(w, Nat.2).second = Nat.2)
        (cycle_offset_vertex(n, Pair.new(w, Nat.2)) = (w + n + Nat.2 - Nat.1).mod(n))
        (w.suc = w + Nat.1)
        (w.suc + n = w + n + Nat.1)
        (Nat.1 + Nat.1 = Nat.2)
        ((w + n + Nat.1) + Nat.1 = w + n + Nat.2)
        (w + n + Nat.2 = (w.suc + n) + Nat.1)
        ((w.suc + n) + Nat.1 - Nat.1 = w.suc + n)
        (w + n + Nat.2 - Nat.1 = w.suc + n)
        add_mod_self(w.suc, n)
        (w.suc + n).mod(n) = w.suc.mod(n)
        cycle_offset_vertex(n, Pair.new(w, Nat.2)) = w.suc.mod(n)
    }
}

/// The offset that recovers the predecessor of a vertex.
///
/// Stated through the successor rather than through a subtraction: if `v + 1` reduces to `w`
/// and `v` is below the modulus, the zero offset at `w` returns `v`. The wrap case, where
/// `v + 1` is the modulus and `w` is zero, is where adding a whole turn earns its keep.
theorem cycle_offset_pred(n: Nat, v: Nat, w: Nat) {
    v < n and v.suc.mod(n) = w implies cycle_offset_vertex(n, Pair.new(w, Nat.0)) = v
} by {
    if v < n and v.suc.mod(n) = w {
        (Pair.new(w, Nat.0).first = w)
        (Pair.new(w, Nat.0).second = Nat.0)
        (cycle_offset_vertex(n, Pair.new(w, Nat.0)) = (w + n + Nat.0 - Nat.1).mod(n))
        (w + n + Nat.0 = w + n)
        suc_mod_cases(v, n, w)
        (w = v.suc or (v.suc = n and w = Nat.0))
        if w = v.suc {
            (w + n = (v + n) + Nat.1)
            ((v + n) + Nat.1 - Nat.1 = v + n)
            (w + n - Nat.1 = v + n)
            small_add_mod_self(v, n)
            (v + n).mod(n) = v
            cycle_offset_vertex(n, Pair.new(w, Nat.0)) = v
        }
        if v.suc = n and w = Nat.0 {
            (w + n - Nat.1 = n - Nat.1)
            (n - Nat.1 = v)
            small_mod(v, n)
            v.mod(n) = v
            cycle_offset_vertex(n, Pair.new(w, Nat.0)) = v
        }
        cycle_offset_vertex(n, Pair.new(w, Nat.0)) = v
    }
}

/// A dominated vertex of the cycle is reached from its dominator by one of three offsets.
theorem cycle_dominated_is_offset(n: Nat, d: FiniteSet[Nat], v: Nat) {
    n != Nat.0 and v < n and d.subset_eq(range_set(n))
        and is_dominated(cycle_graph(n), d, v)
        implies has_preimage_in(finite_set_product(d, range_set(Nat.3)),
            cycle_offset_vertex(n), v)
} by {
    if n != Nat.0 and v < n and d.subset_eq(range_set(n))
        and is_dominated(cycle_graph(n), d, v) {
        (is_dominated(cycle_graph(n), d, v)
            = (d.contains(v) or has_neighbor_in(cycle_graph(n), d, v)))
        three_offsets_lt
        (Nat.0 < Nat.3 and Nat.1 < Nat.3 and Nat.2 < Nat.3)
        range_set_contains(Nat.3, Nat.0)
        range_set(Nat.3).contains(Nat.0)
        range_set_contains(Nat.3, Nat.1)
        range_set(Nat.3).contains(Nat.1)
        range_set_contains(Nat.3, Nat.2)
        range_set(Nat.3).contains(Nat.2)
        if d.contains(v) {
            finite_set_product_contains_pair(d, range_set(Nat.3), v, Nat.1)
            finite_set_product(d, range_set(Nat.3)).contains(Pair.new(v, Nat.1))
            cycle_offset_self(n, v)
            cycle_offset_vertex(n, Pair.new(v, Nat.1)) = v
            has_preimage_in_intro(finite_set_product(d, range_set(Nat.3)),
                cycle_offset_vertex(n), v, Pair.new(v, Nat.1))
            has_preimage_in(finite_set_product(d, range_set(Nat.3)),
                cycle_offset_vertex(n), v)
        }
        if has_neighbor_in(cycle_graph(n), d, v) {
            has_neighbor_in_witness(cycle_graph(n), d, v)
            exists(u: Nat) { d.contains(u) and cycle_graph(n).adj(u, v) }
            let (w: Nat) satisfy {
                d.contains(w) and cycle_graph(n).adj(w, v)
            }
            cycle_graph_adj_iff(n, w, v)
            (cycle_graph(n).adj(w, v)
                = (w != v and (w.suc.mod(n) = v or v.suc.mod(n) = w)))
            (w.suc.mod(n) = v or v.suc.mod(n) = w)
            if w.suc.mod(n) = v {
                finite_set_product_contains_pair(d, range_set(Nat.3), w, Nat.2)
                finite_set_product(d, range_set(Nat.3)).contains(Pair.new(w, Nat.2))
                cycle_offset_suc(n, w)
                cycle_offset_vertex(n, Pair.new(w, Nat.2)) = w.suc.mod(n)
                cycle_offset_vertex(n, Pair.new(w, Nat.2)) = v
                has_preimage_in_intro(finite_set_product(d, range_set(Nat.3)),
                    cycle_offset_vertex(n), v, Pair.new(w, Nat.2))
                has_preimage_in(finite_set_product(d, range_set(Nat.3)),
                    cycle_offset_vertex(n), v)
            }
            if v.suc.mod(n) = w {
                finite_set_product_contains_pair(d, range_set(Nat.3), w, Nat.0)
                finite_set_product(d, range_set(Nat.3)).contains(Pair.new(w, Nat.0))
                cycle_offset_pred(n, v, w)
                cycle_offset_vertex(n, Pair.new(w, Nat.0)) = v
                has_preimage_in_intro(finite_set_product(d, range_set(Nat.3)),
                    cycle_offset_vertex(n), v, Pair.new(w, Nat.0))
                has_preimage_in(finite_set_product(d, range_set(Nat.3)),
                    cycle_offset_vertex(n), v)
            }
            has_preimage_in(finite_set_product(d, range_set(Nat.3)),
                cycle_offset_vertex(n), v)
        }
        has_preimage_in(finite_set_product(d, range_set(Nat.3)), cycle_offset_vertex(n), v)
    }
}

/// A dominating set of a cycle covers it three vertices at a time.
theorem cycle_range_covered_by_dominating(n: Nat, d: FiniteSet[Nat]) {
    n != Nat.0 and d.subset_eq(range_set(n))
        and is_dominating_set(cycle_graph(n), range_set(n), d)
        implies is_covered_by_image(range_set(n),
            finite_set_product(d, range_set(Nat.3)), cycle_offset_vertex(n))
} by {
    if n != Nat.0 and d.subset_eq(range_set(n))
        and is_dominating_set(cycle_graph(n), range_set(n), d) {
        forall(v: Nat) {
            if range_set(n).contains(v) {
                range_set_lt(n, v)
                v < n
                is_dominating_set_apply(cycle_graph(n), range_set(n), d, v)
                is_dominated(cycle_graph(n), d, v)
                cycle_dominated_is_offset(n, d, v)
                has_preimage_in(finite_set_product(d, range_set(Nat.3)),
                    cycle_offset_vertex(n), v)
            }
            (range_set(n).contains(v) implies
                has_preimage_in(finite_set_product(d, range_set(Nat.3)),
                    cycle_offset_vertex(n), v))
        }
        is_covered_by_image_intro(range_set(n), finite_set_product(d, range_set(Nat.3)),
            cycle_offset_vertex(n))
        is_covered_by_image(range_set(n), finite_set_product(d, range_set(Nat.3)),
            cycle_offset_vertex(n))
    }
}

/// A dominating set of a cycle has at least a third as many vertices as the cycle.
theorem cycle_dominating_set_card_bound(n: Nat, d: FiniteSet[Nat]) {
    d.subset_eq(range_set(n)) and is_dominating_set(cycle_graph(n), range_set(n), d)
        implies n <= Nat.3 * fs_card(d)
} by {
    if d.subset_eq(range_set(n)) and is_dominating_set(cycle_graph(n), range_set(n), d) {
        if n = Nat.0 {
            Nat.0 <= Nat.3 * fs_card(d)
            n <= Nat.3 * fs_card(d)
        }
        if n != Nat.0 {
            cycle_range_covered_by_dominating(n, d)
            is_covered_by_image(range_set(n), finite_set_product(d, range_set(Nat.3)),
                cycle_offset_vertex(n))
            fs_card_le_of_covered_by_image(range_set(n),
                finite_set_product(d, range_set(Nat.3)), cycle_offset_vertex(n))
            fs_card(range_set(n)) <= fs_card(finite_set_product(d, range_set(Nat.3)))
            fs_card_product(d, range_set(Nat.3))
            (fs_card(finite_set_product(d, range_set(Nat.3)))
                = fs_card(range_set(Nat.3)) * fs_card(d))
            range_set_card(Nat.3)
            fs_card(range_set(Nat.3)) = Nat.3
            range_set_card(n)
            fs_card(range_set(n)) = n
            n <= Nat.3 * fs_card(d)
        }
        n <= Nat.3 * fs_card(d)
    }
}

/// The domination number of a cycle is at least a third of its length.
///
/// The cycle is two-regular, so a vertex dominates exactly three vertices, and a dominating set
/// has to account for all of them. The same bound as for the path, by the same count.
theorem cycle_domination_number_lower_bound(n: Nat) {
    n <= Nat.3 * domination_number(cycle_graph(n), range_set(n))
} by {
    domination_number_attained(cycle_graph(n), range_set(n))
    dominating_size_pred(cycle_graph(n), range_set(n))(
        domination_number(cycle_graph(n), range_set(n)))
    (dominating_size_pred(cycle_graph(n), range_set(n))(
        domination_number(cycle_graph(n), range_set(n))) = exists(d: FiniteSet[Nat]) {
        d.subset_eq(range_set(n)) and is_dominating_set(cycle_graph(n), range_set(n), d)
            and fs_card(d) = domination_number(cycle_graph(n), range_set(n))
    })
    exists(d: FiniteSet[Nat]) {
        d.subset_eq(range_set(n)) and is_dominating_set(cycle_graph(n), range_set(n), d)
            and fs_card(d) = domination_number(cycle_graph(n), range_set(n))
    }
    let (e: FiniteSet[Nat]) satisfy {
        e.subset_eq(range_set(n)) and is_dominating_set(cycle_graph(n), range_set(n), e)
            and fs_card(e) = domination_number(cycle_graph(n), range_set(n))
    }
    cycle_dominating_set_card_bound(n, e)
    n <= Nat.3 * fs_card(e)
    n <= Nat.3 * domination_number(cycle_graph(n), range_set(n))
}
