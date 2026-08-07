from nat import Nat, lte_and_lt
from finite_set import FiniteSet, fs_remove, fs_difference,
    finite_set_difference_contains_eq, finite_set_subset_trans
from data.finite.finite_set_card import fs_card
from data.finite.finite_set_card_compare import fs_card_mono
from data.finite.finite_set_card_difference import fs_card_difference_of_subset
from data.finite.finite_set_card_ops import fs_card_remove_of_contains
from data.finite.finite_set_subset_intro import fs_subset_eq_intro
from data.finite.finite_set_card_members import fs_member_of_card_pos
from data.finite.finite_set_membership import fs_remove_contains_eq, fs_remove_subset
from graph.simple_graph import SimpleGraph, simple_graph_adj_irreflexive, simple_graph_adj_comm
from graph.simple_graph_degree import degree, degree_eq_neighborhood_card, neighborhood,
    neighborhood_contains_eq
from graph.simple_graph_domination import has_neighbor_in, has_neighbor_in_intro,
    has_neighbor_in_witness, is_dominated, is_dominated_of_contains, is_dominated_of_adj,
    is_dominating_set, is_dominating_set_apply, is_dominating_set_intro
from graph.simple_graph_domination_number import domination_number, domination_number_attained,
    domination_number_is_least, dominating_size_pred
from graph.simple_graph_bipartite import has_no_isolated_vertices, has_no_isolated_vertices_apply

numerals Nat

/// True if no vertex can be dropped from `d` and leave it dominating.
///
/// Minimality under inclusion, which is weaker than having the least cardinality: a minimal
/// dominating set need not be a minimum one. It is the right hypothesis for the complement
/// theorem below, which uses only that each vertex is needed for something.
define is_minimal_dominating_set[V](
    g: SimpleGraph[V], s: FiniteSet[V], d: FiniteSet[V]
) -> Bool {
    is_dominating_set(g, s, d) and forall(v: V) {
        d.contains(v) implies not is_dominating_set(g, s, fs_remove(d, v))
    }
}

/// A minimal dominating set is dominating.
theorem is_minimal_dominating_set_dominating[V](
    g: SimpleGraph[V], s: FiniteSet[V], d: FiniteSet[V]
) {
    is_minimal_dominating_set(g, s, d) implies is_dominating_set(g, s, d)
} by {
    if is_minimal_dominating_set(g, s, d) {
        (is_minimal_dominating_set(g, s, d) = (is_dominating_set(g, s, d) and forall(v: V) {
            d.contains(v) implies not is_dominating_set(g, s, fs_remove(d, v))
        }))
        is_dominating_set(g, s, d)
    }
}

/// Every vertex of a minimal dominating set is needed.
theorem is_minimal_dominating_set_apply[V](
    g: SimpleGraph[V], s: FiniteSet[V], d: FiniteSet[V], v: V
) {
    is_minimal_dominating_set(g, s, d) and d.contains(v)
        implies not is_dominating_set(g, s, fs_remove(d, v))
} by {
    if is_minimal_dominating_set(g, s, d) and d.contains(v) {
        (is_minimal_dominating_set(g, s, d) = (is_dominating_set(g, s, d) and forall(u: V) {
            d.contains(u) implies not is_dominating_set(g, s, fs_remove(d, u))
        }))
        forall(u: V) {
            d.contains(u) implies not is_dominating_set(g, s, fs_remove(d, u))
        }
        (d.contains(v) implies not is_dominating_set(g, s, fs_remove(d, v)))
        not is_dominating_set(g, s, fs_remove(d, v))
    }
}

/// A dominating set assembles from the pointwise condition and the minimality condition.
theorem is_minimal_dominating_set_intro[V](
    g: SimpleGraph[V], s: FiniteSet[V], d: FiniteSet[V]
) {
    is_dominating_set(g, s, d) and (forall(v: V) {
        d.contains(v) implies not is_dominating_set(g, s, fs_remove(d, v))
    }) implies is_minimal_dominating_set(g, s, d)
} by {
    if is_dominating_set(g, s, d) and forall(v: V) {
        d.contains(v) implies not is_dominating_set(g, s, fs_remove(d, v))
    } {
        (is_minimal_dominating_set(g, s, d) = (is_dominating_set(g, s, d) and forall(u: V) {
            d.contains(u) implies not is_dominating_set(g, s, fs_remove(d, u))
        }))
        is_minimal_dominating_set(g, s, d)
    }
}

/// Dropping a vertex from a dominating set leaves some vertex of `s` undominated.
///
/// The usable form of minimality: it names the vertex that the dropped one was responsible for.
theorem minimal_dominating_witness[V](
    g: SimpleGraph[V], s: FiniteSet[V], d: FiniteSet[V], v: V
) {
    is_minimal_dominating_set(g, s, d) and d.contains(v)
        implies exists(u: V) { s.contains(u) and not is_dominated(g, fs_remove(d, v), u) }
} by {
    if is_minimal_dominating_set(g, s, d) and d.contains(v) {
        is_minimal_dominating_set_apply(g, s, d, v)
        not is_dominating_set(g, s, fs_remove(d, v))
        if not exists(u: V) { s.contains(u) and not is_dominated(g, fs_remove(d, v), u) } {
            forall(u: V) {
                if s.contains(u) {
                    if not is_dominated(g, fs_remove(d, v), u) {
                        exists(w: V) {
                            s.contains(w) and not is_dominated(g, fs_remove(d, v), w)
                        }
                        false
                    }
                    is_dominated(g, fs_remove(d, v), u)
                }
                (s.contains(u) implies is_dominated(g, fs_remove(d, v), u))
            }
            is_dominating_set_intro(g, s, fs_remove(d, v))
            is_dominating_set(g, s, fs_remove(d, v))
            false
        }
        exists(u: V) { s.contains(u) and not is_dominated(g, fs_remove(d, v), u) }
    }
}

/// A vertex of positive degree has a neighbour in the ambient set.
theorem positive_degree_witness[V](g: SimpleGraph[V], s: FiniteSet[V], v: V) {
    Nat.0 < degree(g, s, v) implies exists(w: V) { s.contains(w) and g.adj(v, w) }
} by {
    if Nat.0 < degree(g, s, v) {
        degree_eq_neighborhood_card(g, s, v)
        degree(g, s, v) = fs_card(neighborhood(g, s, v))
        Nat.1 <= fs_card(neighborhood(g, s, v))
        fs_member_of_card_pos(neighborhood(g, s, v))
        exists(x: V) { neighborhood(g, s, v).contains(x) }
        let (w: V) satisfy {
            neighborhood(g, s, v).contains(w)
        }
        neighborhood_contains_eq(g, s, v, w)
        (s.contains(w) and g.adj(v, w))
        exists(x: V) { s.contains(x) and g.adj(v, x) }
    }
}

/// Every vertex of a minimal dominating set has a neighbour outside it.
///
/// The heart of the complement theorem. Minimality names a vertex `u` that only `v` was
/// dominating. Either `u` is `v` itself, in which case `v` has no neighbour left in `d` and its
/// neighbour outside comes from having positive degree; or `u` is elsewhere, in which case `u`
/// itself lies outside `d` and is adjacent to `v`.
theorem minimal_dominating_neighbor_outside[V](
    g: SimpleGraph[V], s: FiniteSet[V], d: FiniteSet[V], v: V
) {
    has_no_isolated_vertices(g, s) and is_minimal_dominating_set(g, s, d) and s.contains(v)
        and d.contains(v)
        implies has_neighbor_in(g, fs_difference(s, d), v)
} by {
    if has_no_isolated_vertices(g, s) and is_minimal_dominating_set(g, s, d)
        and s.contains(v) and d.contains(v) {
        minimal_dominating_witness(g, s, d, v)
        exists(x: V) { s.contains(x) and not is_dominated(g, fs_remove(d, v), x) }
        let (u: V) satisfy {
            s.contains(u) and not is_dominated(g, fs_remove(d, v), u)
        }
        (is_dominated(g, fs_remove(d, v), u)
            = (fs_remove(d, v).contains(u) or has_neighbor_in(g, fs_remove(d, v), u)))
        not fs_remove(d, v).contains(u)
        not has_neighbor_in(g, fs_remove(d, v), u)
        is_minimal_dominating_set_dominating(g, s, d)
        is_dominating_set(g, s, d)
        is_dominating_set_apply(g, s, d, u)
        is_dominated(g, d, u)
        (is_dominated(g, d, u) = (d.contains(u) or has_neighbor_in(g, d, u)))
        if d.contains(u) {
            fs_remove_contains_eq(d, v, u)
            (fs_remove(d, v).contains(u) = (d.contains(u) and u != v))
            u = v
            has_no_isolated_vertices_apply(g, s, v)
            Nat.0 < degree(g, s, v)
            positive_degree_witness(g, s, v)
            exists(y: V) { s.contains(y) and g.adj(v, y) }
            let (w: V) satisfy {
                s.contains(w) and g.adj(v, w)
            }
            if d.contains(w) {
                simple_graph_adj_irreflexive(g, v)
                not g.adj(v, v)
                w != v
                fs_remove_contains_eq(d, v, w)
                (fs_remove(d, v).contains(w) = (d.contains(w) and w != v))
                fs_remove(d, v).contains(w)
                simple_graph_adj_comm(g, v, w)
                (g.adj(v, w) = g.adj(w, v))
                g.adj(w, v)
                g.adj(w, u)
                has_neighbor_in_intro(g, fs_remove(d, v), u, w)
                has_neighbor_in(g, fs_remove(d, v), u)
                false
            }
            not d.contains(w)
            finite_set_difference_contains_eq(s, d, w)
            (fs_difference(s, d).contains(w) = (s.contains(w) and not d.contains(w)))
            fs_difference(s, d).contains(w)
            simple_graph_adj_comm(g, v, w)
            (g.adj(v, w) = g.adj(w, v))
            g.adj(w, v)
            has_neighbor_in_intro(g, fs_difference(s, d), v, w)
            has_neighbor_in(g, fs_difference(s, d), v)
        }
        if not d.contains(u) {
            has_neighbor_in(g, d, u)
            has_neighbor_in_witness(g, d, u)
            exists(z: V) { d.contains(z) and g.adj(z, u) }
            let (x: V) satisfy {
                d.contains(x) and g.adj(x, u)
            }
            if x != v {
                fs_remove_contains_eq(d, v, x)
                (fs_remove(d, v).contains(x) = (d.contains(x) and x != v))
                fs_remove(d, v).contains(x)
                has_neighbor_in_intro(g, fs_remove(d, v), u, x)
                has_neighbor_in(g, fs_remove(d, v), u)
                false
            }
            x = v
            g.adj(v, u)
            simple_graph_adj_comm(g, v, u)
            (g.adj(v, u) = g.adj(u, v))
            g.adj(u, v)
            finite_set_difference_contains_eq(s, d, u)
            (fs_difference(s, d).contains(u) = (s.contains(u) and not d.contains(u)))
            fs_difference(s, d).contains(u)
            has_neighbor_in_intro(g, fs_difference(s, d), v, u)
            has_neighbor_in(g, fs_difference(s, d), v)
        }
        has_neighbor_in(g, fs_difference(s, d), v)
    }
}

/// The complement of a minimal dominating set is dominating.
///
/// The classical theorem of Ore. A vertex outside `d` lies in the complement and dominates
/// itself; a vertex inside `d` has a neighbour outside by the lemma above. This is what gives
/// the bound of half the vertices for the domination number of a graph without isolated
/// vertices, since the two sets partition `s`.
theorem minimal_dominating_complement_dominates[V](
    g: SimpleGraph[V], s: FiniteSet[V], d: FiniteSet[V]
) {
    has_no_isolated_vertices(g, s) and is_minimal_dominating_set(g, s, d)
        implies is_dominating_set(g, s, fs_difference(s, d))
} by {
    if has_no_isolated_vertices(g, s) and is_minimal_dominating_set(g, s, d) {
        forall(v: V) {
            if s.contains(v) {
                if d.contains(v) {
                    minimal_dominating_neighbor_outside(g, s, d, v)
                    has_neighbor_in(g, fs_difference(s, d), v)
                    (is_dominated(g, fs_difference(s, d), v)
                        = (fs_difference(s, d).contains(v)
                            or has_neighbor_in(g, fs_difference(s, d), v)))
                    is_dominated(g, fs_difference(s, d), v)
                }
                if not d.contains(v) {
                    finite_set_difference_contains_eq(s, d, v)
                    (fs_difference(s, d).contains(v) = (s.contains(v) and not d.contains(v)))
                    fs_difference(s, d).contains(v)
                    is_dominated_of_contains(g, fs_difference(s, d), v)
                    is_dominated(g, fs_difference(s, d), v)
                }
                is_dominated(g, fs_difference(s, d), v)
            }
            (s.contains(v) implies is_dominated(g, fs_difference(s, d), v))
        }
        is_dominating_set_intro(g, s, fs_difference(s, d))
        is_dominating_set(g, s, fs_difference(s, d))
    }
}

/// The difference of a finite set with anything sits inside it.
theorem fs_difference_subset[T](s: FiniteSet[T], d: FiniteSet[T]) {
    fs_difference(s, d).subset_eq(s)
} by {
    forall(x: T) {
        if fs_difference(s, d).contains(x) {
            finite_set_difference_contains_eq(s, d, x)
            (fs_difference(s, d).contains(x) = (s.contains(x) and not d.contains(x)))
            s.contains(x)
        }
        (fs_difference(s, d).contains(x) implies s.contains(x))
    }
    fs_subset_eq_intro(fs_difference(s, d), s)
    fs_difference(s, d).subset_eq(s)
}

/// A dominating subset of least size is minimal under inclusion.
///
/// Dropping a vertex leaves a strictly smaller subset, which cannot be dominating without
/// beating the minimum.
theorem minimum_dominating_is_minimal[V](
    g: SimpleGraph[V], s: FiniteSet[V], d: FiniteSet[V]
) {
    d.subset_eq(s) and is_dominating_set(g, s, d) and fs_card(d) = domination_number(g, s)
        implies is_minimal_dominating_set(g, s, d)
} by {
    if d.subset_eq(s) and is_dominating_set(g, s, d)
        and fs_card(d) = domination_number(g, s) {
        forall(v: V) {
            if d.contains(v) {
                if is_dominating_set(g, s, fs_remove(d, v)) {
                    fs_remove_subset(d, v)
                    fs_remove(d, v).subset_eq(d)
                    finite_set_subset_trans(fs_remove(d, v), d, s)
                    fs_remove(d, v).subset_eq(s)
                    domination_number_is_least(g, s, fs_remove(d, v))
                    domination_number(g, s) <= fs_card(fs_remove(d, v))
                    fs_card_remove_of_contains(d, v)
                    fs_card(d) = fs_card(fs_remove(d, v)) + Nat.1
                    fs_card(fs_remove(d, v)) < fs_card(fs_remove(d, v)) + Nat.1
                    fs_card(fs_remove(d, v)) < fs_card(d)
                    lte_and_lt(domination_number(g, s), fs_card(fs_remove(d, v)), fs_card(d))
                    domination_number(g, s) < fs_card(d)
                    lte_and_lt(fs_card(d), fs_card(d), fs_card(d))
                    fs_card(d) < fs_card(d)
                    false
                }
                not is_dominating_set(g, s, fs_remove(d, v))
            }
            (d.contains(v) implies not is_dominating_set(g, s, fs_remove(d, v)))
        }
        is_minimal_dominating_set_intro(g, s, d)
        is_minimal_dominating_set(g, s, d)
    }
}

/// The domination number of a graph without isolated vertices is at most half the vertices.
///
/// A minimum dominating set and its complement are both dominating, and they partition the
/// vertex set, so neither can be more than half of it. This is the classical bound.
theorem domination_number_double_le_card[V](g: SimpleGraph[V], s: FiniteSet[V]) {
    has_no_isolated_vertices(g, s)
        implies Nat.2 * domination_number(g, s) <= fs_card(s)
} by {
    if has_no_isolated_vertices(g, s) {
        domination_number_attained(g, s)
        dominating_size_pred(g, s)(domination_number(g, s))
        (dominating_size_pred(g, s)(domination_number(g, s)) = exists(e: FiniteSet[V]) {
            e.subset_eq(s) and is_dominating_set(g, s, e)
                and fs_card(e) = domination_number(g, s)
        })
        let (d: FiniteSet[V]) satisfy {
            d.subset_eq(s) and is_dominating_set(g, s, d)
                and fs_card(d) = domination_number(g, s)
        }
        minimum_dominating_is_minimal(g, s, d)
        is_minimal_dominating_set(g, s, d)
        minimal_dominating_complement_dominates(g, s, d)
        is_dominating_set(g, s, fs_difference(s, d))
        fs_difference_subset(s, d)
        fs_difference(s, d).subset_eq(s)
        domination_number_is_least(g, s, fs_difference(s, d))
        domination_number(g, s) <= fs_card(fs_difference(s, d))
        fs_card_difference_of_subset(s, d)
        fs_card(fs_difference(s, d)) = fs_card(s) - fs_card(d)
        domination_number(g, s) <= fs_card(s) - domination_number(g, s)
        fs_card_mono(d, s)
        fs_card(d) <= fs_card(s)
        domination_number(g, s) <= fs_card(s)
        (domination_number(g, s) + domination_number(g, s)
            <= (fs_card(s) - domination_number(g, s)) + domination_number(g, s))
        ((fs_card(s) - domination_number(g, s)) + domination_number(g, s) = fs_card(s))
        (domination_number(g, s) + domination_number(g, s) <= fs_card(s))
        (Nat.2 * domination_number(g, s)
            = domination_number(g, s) + domination_number(g, s))
        Nat.2 * domination_number(g, s) <= fs_card(s)
    }
}
