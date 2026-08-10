from nat import Nat
from pair import Pair
from list import List, singleton_unique
from finite_set import FiniteSet, finite_set_ext_contains
from data.basic.relation_basic import is_equivalence
from data.basic.functions import binary_function_extensionality, binary_function_eq_transport_predicate_rev
from data.finite.finite_set_card import fs_card
from data.list.list_unique_cons import unique_cons_parts
from graph.simple_graph import SimpleGraph, simple_graph_adj_ne
from graph.simple_graph_walks import simple_graph_walk, simple_graph_walk_refl, simple_graph_walk_nil,
    simple_graph_walk_cons_iff, simple_graph_walk_vertices, simple_graph_path, simple_graph_path_intro,
    simple_graph_path_walk, simple_graph_path_vertices_unique, simple_graph_reachable_iff_exists_walk
from graph.simple_graph_connectivity import simple_graph_reachable, simple_graph_reachable_is_equivalence
from graph.simple_graph_tree import walk_split_in_steps, unique_suffix_of_add, unique_cons_intro,
    cons_contains_imp_tail_contains, dup_in_add_not_unique, walk_finish_in_vertex_list,
    is_tree, is_acyclic, simple_graph_connected_on, tree_connected_on, tree_acyclic
from graph.simple_graph_bipartite_odd_cycle import simple_graph_walk_concat, simple_graph_walk_reverse,
    rev_walk_targets, has_odd_cycle, is_bipartite_no_odd_cycle
from graph.simple_graph_degree_sum import degree_sum
from graph.simple_graph_edges import directed_edges, directed_edge_count, directed_edge_count_eq,
    directed_edges_contains_eq, directed_edges_first
from graph.simple_graph_edge_fibers import directed_edges_from_set, directed_edges_from_set_contains_eq
from graph.simple_graph_handshake import handshake_lemma
from graph.simple_graph_forest import forest_edges_bound, forest_tree_of_edge_count
from graph.simple_graph_bipartite import is_bipartite

numerals Nat

// ============================================================================
// (a) The handshake lemma.
// ============================================================================

/// The directed edges leaving the whole vertex set are exactly the directed edges.
theorem directed_edges_from_set_self_eq[V](g: SimpleGraph[V], s: FiniteSet[V]) {
    directed_edges_from_set(g, s, s) = directed_edges(g, s)
} by {
    forall(p: Pair[V, V]) {
        directed_edges_from_set_contains_eq(g, s, s, p)
        directed_edges_from_set(g, s, s).contains(p) =
            (directed_edges(g, s).contains(p) and s.contains(p.first))
        directed_edges_contains_eq(g, s, p)
        directed_edges(g, s).contains(p) =
            (s.contains(p.first) and s.contains(p.second) and g.adj(p.first, p.second))
        if directed_edges_from_set(g, s, s).contains(p) {
            directed_edges(g, s).contains(p)
        }
        if directed_edges(g, s).contains(p) {
            directed_edges_first(g, s, p)
            directed_edges(g, s).contains(p) implies s.contains(p.first)
            s.contains(p.first)
            directed_edges_from_set(g, s, s).contains(p)
        }
        directed_edges_from_set(g, s, s).contains(p) = directed_edges(g, s).contains(p)
    }
    finite_set_ext_contains(directed_edges_from_set(g, s, s), directed_edges(g, s))
}

/// The handshake lemma: the sum of the degrees of the vertices of `s` is the number of
/// ordered adjacent pairs drawn from `s`.
///
/// This is the statement carried by the degree API (`handshake_lemma` in
/// `simple_graph_handshake.ac`), restated here as part of the graph structure file.
theorem graph_handshake_lemma[V](g: SimpleGraph[V], s: FiniteSet[V]) {
    degree_sum(g, s) = fs_card(directed_edges_from_set(g, s, s))
} by {
    handshake_lemma(g, s)
    degree_sum(g, s) = fs_card(directed_edges_from_set(g, s, s))
}

/// The handshake lemma in directed form: the degree sum equals the directed edge count.
theorem graph_handshake_directed[V](g: SimpleGraph[V], s: FiniteSet[V]) {
    degree_sum(g, s) = directed_edge_count(g, s)
} by {
    graph_handshake_lemma(g, s)
    degree_sum(g, s) = fs_card(directed_edges_from_set(g, s, s))
    directed_edges_from_set_self_eq(g, s)
    directed_edges_from_set(g, s, s) = directed_edges(g, s)
    fs_card(directed_edges_from_set(g, s, s)) = fs_card(directed_edges(g, s))
    directed_edge_count_eq(g, s)
    directed_edge_count(g, s) = fs_card(directed_edges(g, s))
    degree_sum(g, s) = directed_edge_count(g, s)
}

// The classical statement "the sum of the degrees is twice the number of edges" needs the
// undirected edge count, which the library takes to be half the directed count:
//   graph_edge_count(g, s) = directed_edge_count(g, s).div(Nat.2).
// It is equivalent to `directed_edge_count(g, s) = Nat.2 * graph_edge_count(g, s)`, i.e.
// to the evenness of the directed edge count. Evenness holds because the directed edges
// come in reversal pairs (x, y) and (y, x), but the pairing argument (a fixed-point-free
// involution on a finite set has even cardinality) is not yet in the library, so the
// statement is left open here. The provable forms above, together with the original
// `handshake_lemma`, are the deliverable for the statement.
// theorem graph_handshake_edge_count[V](g: SimpleGraph[V], s: FiniteSet[V]) {
//     degree_sum(g, s) = Nat.2 * graph_edge_count(g, s)
// }

// ============================================================================
// (b) A walk between two vertices contains a path between them.
// ============================================================================

/// Every walk contains a path between its endpoints: a walk with a repeated vertex can be
/// shortened by cutting out the loop between two occurrences of the vertex.
///
/// The proof walks down the target list. For a walk `u -> head -> ... -> v`, the induction
/// hypothesis supplies a path from `head` to `v`. If that path already passes through `u`,
/// cut it at the first re-occurrence of `u` and keep the suffix (its vertices are distinct
/// because they are a subsequence of a path, and `u` does not reappear inside it);
/// otherwise prepend the first edge, which stays a path because `u` is fresh.
theorem walk_has_path[V](g: SimpleGraph[V], a: V, b: V, steps: List[V]) {
    simple_graph_walk(g, a, b, steps) implies exists(steps2: List[V]) {
        simple_graph_path(g, a, b, steps2)
    }
} by {
    define pc(xs: List[V], u: V, v: V) -> Bool {
        simple_graph_walk(g, u, v, xs) implies exists(ys: List[V]) {
            simple_graph_path(g, u, v, ys)
        }
    }

    define p(xs: List[V]) -> Bool {
        forall(u: V, v: V) {
            pc(xs, u, v)
        }
    }

    forall(u: V, v: V) {
        if simple_graph_walk(g, u, v, List.nil[V]) {
            simple_graph_walk_nil(g, u, v)
            u = v
            simple_graph_walk_vertices(u, List.nil[V]) = List.cons(u, List.nil[V])
            List.cons(u, List.nil[V]) = List.singleton(u)
            singleton_unique(u)
            List.singleton(u).is_unique
            simple_graph_walk_vertices(u, List.nil[V]).is_unique
            simple_graph_walk(g, u, v, List.nil[V])
            simple_graph_path_intro(g, u, v, List.nil[V])
            simple_graph_path(g, u, v, List.nil[V])
            exists(ys: List[V]) {
                ys = List.nil[V] and simple_graph_path(g, u, v, ys)
            }
        }
        pc(List.nil[V], u, v) = (simple_graph_walk(g, u, v, List.nil[V]) implies exists(ys: List[V]) {
            simple_graph_path(g, u, v, ys)
        })
        pc(List.nil[V], u, v)
    }
    p(List.nil[V])

    forall(head: V, tail: List[V]) {
        if p(tail) {
            forall(u: V, v: V) {
                if simple_graph_walk(g, u, v, List.cons(head, tail)) {
                    simple_graph_walk_cons_iff(g, u, v, head, tail)
                    simple_graph_walk(g, u, v, List.cons(head, tail)) =
                        (g.adj(u, head) and simple_graph_walk(g, head, v, tail))
                    g.adj(u, head)
                    simple_graph_walk(g, head, v, tail)
                    pc(tail, head, v)
                    pc(tail, head, v) = (simple_graph_walk(g, head, v, tail) implies exists(ys: List[V]) {
                        simple_graph_path(g, head, v, ys)
                    })
                    exists(ys: List[V]) {
                        simple_graph_path(g, head, v, ys)
                    }
                    let w: List[V] satisfy {
                        simple_graph_path(g, head, v, w)
                    }
                    if List.cons(head, w).contains(u) {
                        simple_graph_adj_ne(g, u, head)
                        g.adj(u, head) implies u != head
                        u != head
                        cons_contains_imp_tail_contains(head, w, u)
                        List.cons(head, w).contains(u) and u != head implies w.contains(u)
                        w.contains(u)
                        simple_graph_path_walk(g, head, v, w)
                        simple_graph_path(g, head, v, w) implies simple_graph_walk(g, head, v, w)
                        simple_graph_walk(g, head, v, w)
                        walk_split_in_steps(g, head, u, v, w)
                        (simple_graph_walk(g, head, v, w) and w.contains(u)) implies exists(pre: List[V], suf: List[V]) {
                            w = pre + suf and simple_graph_walk(g, head, u, pre) and
                                simple_graph_walk(g, u, v, suf)
                        }
                        exists(pre: List[V], suf: List[V]) {
                            w = pre + suf and simple_graph_walk(g, head, u, pre) and
                                simple_graph_walk(g, u, v, suf)
                        }
                        let (pre: List[V], suf: List[V]) satisfy {
                            w = pre + suf and simple_graph_walk(g, head, u, pre) and
                                simple_graph_walk(g, u, v, suf)
                        }
                        simple_graph_path_vertices_unique(g, head, v, w)
                        simple_graph_path(g, head, v, w) implies simple_graph_walk_vertices(head, w).is_unique
                        simple_graph_walk_vertices(head, w).is_unique
                        simple_graph_walk_vertices(head, w) = List.cons(head, w)
                        List.cons(head, w).is_unique
                        w = pre + suf
                        List.cons(head, w) = List.cons(head, pre + suf)
                        List.cons(head, pre + suf) = List.cons(head, pre) + suf
                        unique_cons_parts(head, pre + suf)
                        List.cons(head, pre + suf).is_unique implies (pre + suf).is_unique
                        (pre + suf).is_unique
                        unique_suffix_of_add(pre, suf)
                        (pre + suf).is_unique implies suf.is_unique
                        suf.is_unique
                        walk_finish_in_vertex_list(g, head, u, pre)
                        simple_graph_walk(g, head, u, pre) implies List.cons(head, pre).contains(u)
                        List.cons(head, pre).contains(u)
                        cons_contains_imp_tail_contains(head, pre, u)
                        List.cons(head, pre).contains(u) and u != head implies pre.contains(u)
                        pre.contains(u)
                        if suf.contains(u) {
                            dup_in_add_not_unique(pre, suf, u)
                            pre.contains(u) and suf.contains(u) implies not (pre + suf).is_unique
                            not (pre + suf).is_unique
                            false
                        }
                        not suf.contains(u)
                        unique_cons_intro(u, suf)
                        not suf.contains(u) and suf.is_unique implies List.cons(u, suf).is_unique
                        List.cons(u, suf).is_unique
                        simple_graph_walk_vertices(u, suf) = List.cons(u, suf)
                        simple_graph_walk_vertices(u, suf).is_unique
                        simple_graph_walk(g, u, v, suf)
                        simple_graph_path_intro(g, u, v, suf)
                        simple_graph_path(g, u, v, suf)
                        exists(ys: List[V]) {
                            simple_graph_path(g, u, v, ys)
                        }
                    }
                    if not List.cons(head, w).contains(u) {
                        simple_graph_path_vertices_unique(g, head, v, w)
                        simple_graph_path(g, head, v, w) implies simple_graph_walk_vertices(head, w).is_unique
                        simple_graph_walk_vertices(head, w).is_unique
                        simple_graph_walk_vertices(head, w) = List.cons(head, w)
                        List.cons(head, w).is_unique
                        unique_cons_intro(u, List.cons(head, w))
                        not List.cons(head, w).contains(u) and List.cons(head, w).is_unique implies List.cons(u, List.cons(head, w)).is_unique
                        List.cons(u, List.cons(head, w)).is_unique
                        simple_graph_path_walk(g, head, v, w)
                        simple_graph_path(g, head, v, w) implies simple_graph_walk(g, head, v, w)
                        simple_graph_walk(g, head, v, w)
                        simple_graph_walk_cons_iff(g, u, v, head, w)
                        simple_graph_walk(g, u, v, List.cons(head, w)) =
                            (g.adj(u, head) and simple_graph_walk(g, head, v, w))
                        g.adj(u, head) and simple_graph_walk(g, head, v, w) implies simple_graph_walk(g, u, v, List.cons(head, w))
                        simple_graph_walk(g, u, v, List.cons(head, w))
                        simple_graph_walk_vertices(u, List.cons(head, w)) = List.cons(u, List.cons(head, w))
                        simple_graph_walk_vertices(u, List.cons(head, w)).is_unique
                        simple_graph_path_intro(g, u, v, List.cons(head, w))
                        simple_graph_path(g, u, v, List.cons(head, w))
                        exists(ys: List[V]) {
                            simple_graph_path(g, u, v, ys)
                        }
                    }
                    exists(ys: List[V]) {
                        simple_graph_path(g, u, v, ys)
                    }
                }
                pc(List.cons(head, tail), u, v)
            }
            p(List.cons(head, tail))
        }
    }
    forall(head: V, tail: List[V]) { p(tail) implies p(List.cons(head, tail)) }
    function[T0](x0: List[T0] -> Bool) {
        not forall(x1: T0, x2: List[T0]) { not x0(x2) or x0(List.cons(x1, x2)) } or
        not x0(List.nil[T0]) or forall(x3: List[T0]) { x0(x3) } = true
    }[V](p)
    function[T0](x0: List[T0] -> Bool, x1: List[T0]) {
        not forall(x2: T0, x3: List[T0]) { not x0(x3) or x0(List.cons(x2, x3)) } or
        not x0(List.nil[T0]) or x0(x1)
    }[V](p, steps)
    p(steps)
    pc(steps, a, b)
    if simple_graph_walk(g, a, b, steps) {
        pc(steps, a, b) = (simple_graph_walk(g, a, b, steps) implies exists(ys: List[V]) {
            simple_graph_path(g, a, b, ys)
        })
        exists(ys: List[V]) {
            simple_graph_path(g, a, b, ys)
        }
    }
}

/// A graph with a walk between two vertices has a path between them.
theorem walk_imp_path[V](g: SimpleGraph[V], a: V, b: V) {
    exists(steps: List[V]) { simple_graph_walk(g, a, b, steps) } implies
    exists(steps: List[V]) { simple_graph_path(g, a, b, steps) }
} by {
    if exists(steps: List[V]) { simple_graph_walk(g, a, b, steps) } {
        let steps: List[V] satisfy {
            simple_graph_walk(g, a, b, steps)
        }
        walk_has_path(g, a, b, steps)
        simple_graph_walk(g, a, b, steps) implies exists(steps2: List[V]) {
            simple_graph_path(g, a, b, steps2)
        }
        exists(steps2: List[V]) {
            simple_graph_path(g, a, b, steps2)
        }
    }
}

// ============================================================================
// (c) Connectedness is an equivalence relation on the vertices.
// ============================================================================

/// True when the two vertices are joined by a walk.
define walk_connected[V](g: SimpleGraph[V], x: V, y: V) -> Bool {
    exists(steps: List[V]) {
        simple_graph_walk(g, x, y, steps)
    }
}

/// Every vertex is walk-connected to itself: the length-zero walk.
theorem walk_connected_refl[V](g: SimpleGraph[V], x: V) {
    walk_connected(g, x, x)
} by {
    simple_graph_walk_refl(g, x)
    simple_graph_walk(g, x, x, List.nil[V])
    walk_connected(g, x, x) = exists(steps: List[V]) {
        simple_graph_walk(g, x, x, steps)
    }
    exists(steps: List[V]) {
        steps = List.nil[V] and simple_graph_walk(g, x, x, steps)
    }
    walk_connected(g, x, x)
}

/// Walk-connectedness is symmetric: reversing a walk gives a walk back.
theorem walk_connected_symmetric[V](g: SimpleGraph[V], x: V, y: V) {
    walk_connected(g, x, y) implies walk_connected(g, y, x)
} by {
    if walk_connected(g, x, y) {
        walk_connected(g, x, y) = exists(steps: List[V]) {
            simple_graph_walk(g, x, y, steps)
        }
        exists(steps: List[V]) {
            simple_graph_walk(g, x, y, steps)
        }
        let steps: List[V] satisfy {
            simple_graph_walk(g, x, y, steps)
        }
        simple_graph_walk_reverse(g, x, y, steps)
        simple_graph_walk(g, x, y, steps) implies simple_graph_walk(g, y, x, rev_walk_targets(steps, x))
        simple_graph_walk(g, y, x, rev_walk_targets(steps, x))
        walk_connected(g, y, x) = exists(steps2: List[V]) {
            simple_graph_walk(g, y, x, steps2)
        }
        exists(steps2: List[V]) {
            steps2 = rev_walk_targets(steps, x) and simple_graph_walk(g, y, x, steps2)
        }
        walk_connected(g, y, x)
    }
}

/// Walk-connectedness is transitive: concatenating the two walks gives a walk.
theorem walk_connected_transitive[V](g: SimpleGraph[V], x: V, y: V, z: V) {
    walk_connected(g, x, y) and walk_connected(g, y, z) implies walk_connected(g, x, z)
} by {
    if walk_connected(g, x, y) and walk_connected(g, y, z) {
        walk_connected(g, x, y) = exists(steps: List[V]) {
            simple_graph_walk(g, x, y, steps)
        }
        exists(steps: List[V]) {
            simple_graph_walk(g, x, y, steps)
        }
        let w1: List[V] satisfy {
            simple_graph_walk(g, x, y, w1)
        }
        walk_connected(g, y, z) = exists(steps: List[V]) {
            simple_graph_walk(g, y, z, steps)
        }
        exists(steps: List[V]) {
            simple_graph_walk(g, y, z, steps)
        }
        let w2: List[V] satisfy {
            simple_graph_walk(g, y, z, w2)
        }
        simple_graph_walk_concat(g, x, y, z, w1, w2)
        simple_graph_walk(g, x, y, w1) and simple_graph_walk(g, y, z, w2) implies simple_graph_walk(g, x, z, w1 + w2)
        simple_graph_walk(g, x, z, w1 + w2)
        walk_connected(g, x, z) = exists(steps: List[V]) {
            simple_graph_walk(g, x, z, steps)
        }
        exists(steps: List[V]) {
            steps = w1 + w2 and simple_graph_walk(g, x, z, steps)
        }
        walk_connected(g, x, z)
    }
}

/// Walk-connectedness is exactly reachability: each side is the existence of a walk.
theorem walk_connected_iff_reachable[V](g: SimpleGraph[V], x: V, y: V) {
    walk_connected(g, x, y) = simple_graph_reachable(g, x, y)
} by {
    simple_graph_reachable_iff_exists_walk(g, x, y)
    simple_graph_reachable(g, x, y) = exists(steps: List[V]) {
        simple_graph_walk(g, x, y, steps)
    }
    walk_connected(g, x, y) = exists(steps: List[V]) {
        simple_graph_walk(g, x, y, steps)
    }
    walk_connected(g, x, y) = simple_graph_reachable(g, x, y)
}

/// Walk-connectedness is exactly reachability, as a relation on the vertices.
theorem walk_connected_eq_reachable[V](g: SimpleGraph[V]) {
    walk_connected(g) = simple_graph_reachable(g)
} by {
    forall(x: V, y: V) {
        walk_connected_iff_reachable(g, x, y)
        walk_connected(g, x, y) = simple_graph_reachable(g, x, y)
    }
    binary_function_extensionality(walk_connected(g), simple_graph_reachable(g))
    walk_connected(g) = simple_graph_reachable(g)
}

/// Connectedness is an equivalence relation on the vertices.
///
/// The three walk-based properties (reflexive, symmetric, transitive) are proved
/// individually above; this packages them as an equivalence relation by transporting the
/// equivalence of the reachability relation (which walk-connectedness equals pointwise).
theorem walk_connected_is_equivalence[V](g: SimpleGraph[V]) {
    is_equivalence(walk_connected(g))
} by {
    walk_connected_eq_reachable(g)
    walk_connected(g) = simple_graph_reachable(g)
    simple_graph_reachable_is_equivalence(g)
    is_equivalence(simple_graph_reachable(g))
    binary_function_eq_transport_predicate_rev(is_equivalence[V], walk_connected(g), simple_graph_reachable(g))
    walk_connected(g) = simple_graph_reachable(g) and is_equivalence(simple_graph_reachable(g)) implies is_equivalence(walk_connected(g))
    is_equivalence(walk_connected(g))
}

/// Reachability is an equivalence relation on the vertices; its classes are the connected
/// components.
theorem reachable_is_equivalence[V](g: SimpleGraph[V]) {
    is_equivalence(simple_graph_reachable(g))
} by {
    simple_graph_reachable_is_equivalence(g)
    is_equivalence(simple_graph_reachable(g))
}

// ============================================================================
// (d) Trees: every tree on `n` vertices has `n - 1` edges.
// ============================================================================

/// A tree on `s` is connected on `s`.
theorem graph_tree_connected[V](g: SimpleGraph[V], s: FiniteSet[V]) {
    is_tree(g, s) implies simple_graph_connected_on(g, s)
} by {
    tree_connected_on(g, s)
    is_tree(g, s) implies simple_graph_connected_on(g, s)
}

/// A tree on `s` is acyclic on `s`.
theorem graph_tree_acyclic[V](g: SimpleGraph[V], s: FiniteSet[V]) {
    is_tree(g, s) implies is_acyclic(g, s)
} by {
    tree_acyclic(g, s)
}

/// A forest on `n` vertices with `n - 1` edges is a tree: the edge count forces
/// connectedness.
///
/// The directed edge count counts each undirected edge twice, once per orientation, so
/// `n - 1` undirected edges appear as `2 n - 2` directed edges.
theorem graph_tree_of_edge_count[V](g: SimpleGraph[V], s: FiniteSet[V]) {
    is_acyclic(g, s) and Nat.1 <= fs_card(s) and
        directed_edge_count(g, s) = Nat.2 * fs_card(s) - Nat.2
        implies simple_graph_connected_on(g, s)
} by {
    forest_tree_of_edge_count(g, s)
    is_acyclic(g, s) and Nat.1 <= fs_card(s) and directed_edge_count(g, s) = Nat.2 * fs_card(s) - Nat.2 implies simple_graph_connected_on(g, s)
}

/// A forest on `s` has at most `2 |s| - 2` directed edges, i.e. at most `|s| - 1`
/// undirected edges.
theorem graph_forest_edges_bound[V](g: SimpleGraph[V], s: FiniteSet[V]) {
    is_acyclic(g, s) implies directed_edge_count(g, s) <= Nat.2 * fs_card(s) - Nat.2
} by {
    forest_edges_bound(g, s)
    is_acyclic(g, s) implies directed_edge_count(g, s) <= Nat.2 * fs_card(s) - Nat.2
}

// The converse — every tree on `n` vertices has exactly `n - 1` edges, i.e.
//   is_tree(g, s) implies directed_edge_count(g, s) = Nat.2 * fs_card(s) - Nat.2 —
// is open in the tree work. The proof peels off a leaf (`forest_has_leaf` together with
// `edges_remove_leaf`), but keeping the remaining graph connected after the removal is not
// yet a standalone lemma there, so the statement is commented out per AGENTS.md. The
// verified results above (`graph_tree_of_edge_count`, `graph_forest_edges_bound`) are the
// existing edge-count part of the tree work.
// theorem graph_tree_edge_count[V](g: SimpleGraph[V], s: FiniteSet[V]) {
//     is_tree(g, s) implies directed_edge_count(g, s) = Nat.2 * fs_card(s) - Nat.2
// }

// ============================================================================
// (e) A graph is bipartite if and only if it has no odd cycle.
// ============================================================================

/// A bipartite graph has no odd cycle: every closed walk alternates between the two sides,
/// so its length is even.
theorem graph_bipartite_no_odd_cycle[V](g: SimpleGraph[V], s: FiniteSet[V]) {
    is_bipartite(g, s) implies not has_odd_cycle(g, s)
} by {
    is_bipartite_no_odd_cycle(g, s)
    is_bipartite(g, s) implies not has_odd_cycle(g, s)
}

// The converse — a graph with no odd cycle is bipartite — needs the lemma that an odd
// closed walk contains an odd cycle (cut the walk at a repeated vertex and keep the
// shorter odd piece). That extraction is left open in `simple_graph_bipartite_odd_cycle.ac`,
// so the iff is stated here as a comment rather than a theorem. The verified direction
// above is the forward half of the equivalence.
// theorem graph_bipartite_iff_no_odd_cycle[V](g: SimpleGraph[V], s: FiniteSet[V]) {
//     is_bipartite(g, s) = not has_odd_cycle(g, s)
// }
