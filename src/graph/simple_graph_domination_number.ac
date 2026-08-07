from nat import Nat, is_min, has_min, is_min_apply, is_min_false_below, false_below,
    false_below_apply
from finite_set import FiniteSet, finite_set_subset_refl
from data.finite.finite_set_card import fs_card
from graph.simple_graph import SimpleGraph
from graph.simple_graph_domination import is_dominating_set, is_dominating_set_self

numerals Nat

/// True of the sizes achieved by dominating subsets of `s`.
///
/// Packaged as a predicate on `Nat` so the minimum construction can be applied to it.
define dominating_size_pred[V](g: SimpleGraph[V], s: FiniteSet[V]) -> (Nat -> Bool) {
    function(n: Nat) {
        exists(d: FiniteSet[V]) {
            d.subset_eq(s) and is_dominating_set(g, s, d) and fs_card(d) = n
        }
    }
}

/// A dominating subset achieves its own size.
theorem dominating_size_pred_intro[V](
    g: SimpleGraph[V], s: FiniteSet[V], d: FiniteSet[V]
) {
    d.subset_eq(s) and is_dominating_set(g, s, d) implies dominating_size_pred(g, s)(fs_card(d))
} by {
    if d.subset_eq(s) and is_dominating_set(g, s, d) {
        dominating_size_pred(g, s)(fs_card(d)) = exists(e: FiniteSet[V]) {
            e.subset_eq(s) and is_dominating_set(g, s, e) and fs_card(e) = fs_card(d)
        }
        exists(e: FiniteSet[V]) {
            e.subset_eq(s) and is_dominating_set(g, s, e) and fs_card(e) = fs_card(d)
        }
        dominating_size_pred(g, s)(fs_card(d))
    }
}

/// The ambient set itself achieves a dominating size.
theorem dominating_size_pred_ambient[V](g: SimpleGraph[V], s: FiniteSet[V]) {
    dominating_size_pred(g, s)(fs_card(s))
} by {
    finite_set_subset_refl(s)
    s.subset_eq(s)
    is_dominating_set_self(g, s)
    is_dominating_set(g, s, s)
    dominating_size_pred_intro(g, s, s)
    dominating_size_pred(g, s)(fs_card(s))
}

/// The fewest vertices in a dominating subset of `s`.
///
/// Well defined because the ambient set dominates itself, so some dominating size
/// exists, and every nonempty set of naturals has a least element.
let domination_number[V](g: SimpleGraph[V], s: FiniteSet[V]) -> result: Nat satisfy {
    is_min(dominating_size_pred(g, s), result)
} by {
    dominating_size_pred_ambient(g, s)
    has_min(dominating_size_pred(g, s), fs_card(s))
}

/// Some dominating subset attains the domination number.
theorem domination_number_attained[V](g: SimpleGraph[V], s: FiniteSet[V]) {
    dominating_size_pred(g, s)(domination_number(g, s))
} by {
    is_min(dominating_size_pred(g, s), domination_number(g, s))
    is_min_apply(dominating_size_pred(g, s), domination_number(g, s))
}

/// No dominating subset is smaller than the domination number.
theorem domination_number_is_least[V](
    g: SimpleGraph[V], s: FiniteSet[V], d: FiniteSet[V]
) {
    d.subset_eq(s) and is_dominating_set(g, s, d) implies domination_number(g, s) <= fs_card(d)
} by {
    if d.subset_eq(s) and is_dominating_set(g, s, d) {
        dominating_size_pred_intro(g, s, d)
        dominating_size_pred(g, s)(fs_card(d))
        is_min(dominating_size_pred(g, s), domination_number(g, s))
        is_min_false_below(dominating_size_pred(g, s), domination_number(g, s))
        false_below(dominating_size_pred(g, s), domination_number(g, s))
        if fs_card(d) < domination_number(g, s) {
            false_below_apply(dominating_size_pred(g, s), domination_number(g, s), fs_card(d))
            not dominating_size_pred(g, s)(fs_card(d))
            false
        }
        domination_number(g, s) <= fs_card(d)
    }
}

/// The domination number is at most the number of vertices.
theorem domination_number_le_ambient[V](g: SimpleGraph[V], s: FiniteSet[V]) {
    domination_number(g, s) <= fs_card(s)
} by {
    finite_set_subset_refl(s)
    s.subset_eq(s)
    is_dominating_set_self(g, s)
    is_dominating_set(g, s, s)
    domination_number_is_least(g, s, s)
    domination_number(g, s) <= fs_card(s)
}
