from nat import Nat
from finite_set import FiniteSet, finite_set_subset_contains
from data.finite.finite_set_filter import finite_set_filter, finite_set_filter_contains_eq,
    finite_set_filter_subset
from graph.simple_graph import SimpleGraph, simple_graph_adj_comm, simple_graph_adj_ne
from graph.simple_graph_degree import neighborhood, neighborhood_contains_eq, neighbor_pred

numerals Nat

/// True if every edge of `g` inside `s` has an endpoint in `c`.
define is_vertex_cover[V](g: SimpleGraph[V], s: FiniteSet[V], c: FiniteSet[V]) -> Bool {
    forall(x: V, y: V) {
        s.contains(x) and s.contains(y) and g.adj(x, y) implies c.contains(x) or c.contains(y)
    }
}

/// An edge inside the ambient set meets the cover.
theorem is_vertex_cover_apply[V](
    g: SimpleGraph[V], s: FiniteSet[V], c: FiniteSet[V], x: V, y: V
) {
    is_vertex_cover(g, s, c) and s.contains(x) and s.contains(y) and g.adj(x, y) implies c.contains(x) or c.contains(y)
} by {
    if is_vertex_cover(g, s, c) and s.contains(x) and s.contains(y) and g.adj(x, y) {
        is_vertex_cover(g, s, c) = forall(u: V, w: V) {
            s.contains(u) and s.contains(w) and g.adj(u, w) implies c.contains(u) or c.contains(w)
        }
        forall(u: V, w: V) {
            s.contains(u) and s.contains(w) and g.adj(u, w) implies c.contains(u) or c.contains(w)
        }
        s.contains(x) and s.contains(y) and g.adj(x, y) implies c.contains(x) or c.contains(y)
        c.contains(x) or c.contains(y)
    }
}

/// A pointwise covering condition is a vertex cover.
theorem is_vertex_cover_intro[V](g: SimpleGraph[V], s: FiniteSet[V], c: FiniteSet[V]) {
    (forall(x: V, y: V) {
        s.contains(x) and s.contains(y) and g.adj(x, y) implies c.contains(x) or c.contains(y)
    }) implies is_vertex_cover(g, s, c)
} by {
    if forall(x: V, y: V) {
        s.contains(x) and s.contains(y) and g.adj(x, y) implies c.contains(x) or c.contains(y)
    } {
        is_vertex_cover(g, s, c) = forall(u: V, w: V) {
            s.contains(u) and s.contains(w) and g.adj(u, w) implies c.contains(u) or c.contains(w)
        }
        is_vertex_cover(g, s, c)
    }
}

/// The whole ambient set is a vertex cover.
theorem is_vertex_cover_self[V](g: SimpleGraph[V], s: FiniteSet[V]) {
    is_vertex_cover(g, s, s)
} by {
    forall(x: V, y: V) {
        if s.contains(x) and s.contains(y) and g.adj(x, y) {
            s.contains(x) or s.contains(y)
        }
        s.contains(x) and s.contains(y) and g.adj(x, y) implies s.contains(x) or s.contains(y)
    }
    is_vertex_cover_intro(g, s, s)
    is_vertex_cover(g, s, s)
}

/// Enlarging a vertex cover keeps it a vertex cover.
theorem is_vertex_cover_of_subset[V](
    g: SimpleGraph[V], s: FiniteSet[V], c: FiniteSet[V], d: FiniteSet[V]
) {
    c.subset_eq(d) and is_vertex_cover(g, s, c) implies is_vertex_cover(g, s, d)
} by {
    if c.subset_eq(d) and is_vertex_cover(g, s, c) {
        forall(x: V, y: V) {
            if s.contains(x) and s.contains(y) and g.adj(x, y) {
                is_vertex_cover_apply(g, s, c, x, y)
                c.contains(x) or c.contains(y)
                if c.contains(x) {
                    finite_set_subset_contains(c, d, x)
                    d.contains(x)
                    d.contains(x) or d.contains(y)
                }
                if not c.contains(x) {
                    c.contains(y)
                    finite_set_subset_contains(c, d, y)
                    d.contains(y)
                    d.contains(x) or d.contains(y)
                }
                d.contains(x) or d.contains(y)
            }
            s.contains(x) and s.contains(y) and g.adj(x, y) implies d.contains(x) or d.contains(y)
        }
        is_vertex_cover_intro(g, s, d)
        is_vertex_cover(g, s, d)
    }
}

/// Shrinking the ambient set keeps a vertex cover.
theorem is_vertex_cover_of_smaller_ambient[V](
    g: SimpleGraph[V], s: FiniteSet[V], t: FiniteSet[V], c: FiniteSet[V]
) {
    t.subset_eq(s) and is_vertex_cover(g, s, c) implies is_vertex_cover(g, t, c)
} by {
    if t.subset_eq(s) and is_vertex_cover(g, s, c) {
        forall(x: V, y: V) {
            if t.contains(x) and t.contains(y) and g.adj(x, y) {
                finite_set_subset_contains(t, s, x)
                finite_set_subset_contains(t, s, y)
                s.contains(x)
                s.contains(y)
                is_vertex_cover_apply(g, s, c, x, y)
                c.contains(x) or c.contains(y)
            }
            t.contains(x) and t.contains(y) and g.adj(x, y) implies c.contains(x) or c.contains(y)
        }
        is_vertex_cover_intro(g, t, c)
        is_vertex_cover(g, t, c)
    }
}

/// The vertices of `s` adjacent to both `u` and `v`.
define common_neighborhood[V](
    g: SimpleGraph[V], s: FiniteSet[V], u: V, v: V
) -> FiniteSet[V] {
    finite_set_filter(neighborhood(g, s, u), neighbor_pred(g, v))
}

/// A vertex is a common neighbor exactly when it is adjacent to both.
theorem common_neighborhood_contains_eq[V](
    g: SimpleGraph[V], s: FiniteSet[V], u: V, v: V, w: V
) {
    common_neighborhood(g, s, u, v).contains(w) = (s.contains(w) and g.adj(u, w) and g.adj(v, w))
} by {
    finite_set_filter_contains_eq(neighborhood(g, s, u), neighbor_pred(g, v), w)
    neighbor_pred(g, v)(w) = g.adj(v, w)
    common_neighborhood(g, s, u, v).contains(w) = (neighborhood(g, s, u).contains(w) and g.adj(v, w))
    neighborhood_contains_eq(g, s, u, w)
    neighborhood(g, s, u).contains(w) = (s.contains(w) and g.adj(u, w))
}

/// Common neighborhoods sit inside the neighborhood of each vertex.
theorem common_neighborhood_subset[V](g: SimpleGraph[V], s: FiniteSet[V], u: V, v: V) {
    common_neighborhood(g, s, u, v).subset_eq(neighborhood(g, s, u))
} by {
    finite_set_filter_subset(neighborhood(g, s, u), neighbor_pred(g, v))
}

/// A common neighbor of two vertices is adjacent to the first.
theorem common_neighborhood_adj_left[V](
    g: SimpleGraph[V], s: FiniteSet[V], u: V, v: V, w: V
) {
    common_neighborhood(g, s, u, v).contains(w) implies g.adj(u, w)
} by {
    common_neighborhood_contains_eq(g, s, u, v, w)
}

/// A common neighbor of two vertices is adjacent to the second.
theorem common_neighborhood_adj_right[V](
    g: SimpleGraph[V], s: FiniteSet[V], u: V, v: V, w: V
) {
    common_neighborhood(g, s, u, v).contains(w) implies g.adj(v, w)
} by {
    common_neighborhood_contains_eq(g, s, u, v, w)
}
