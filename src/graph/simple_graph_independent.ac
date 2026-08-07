from nat import Nat
from finite_set import FiniteSet, fs_difference, finite_set_subset_contains,
    finite_set_difference_contains_eq
from graph.simple_graph import SimpleGraph, simple_graph_adj_comm, simple_graph_adj_ne
from graph.simple_graph_vertex_sets import is_vertex_cover, is_vertex_cover_intro

numerals Nat

/// True if no two vertices of `t` are adjacent.
define is_independent_in[V](g: SimpleGraph[V], t: FiniteSet[V]) -> Bool {
    forall(x: V, y: V) {
        t.contains(x) and t.contains(y) implies not g.adj(x, y)
    }
}

/// Two members of an independent set are not adjacent.
theorem is_independent_in_apply[V](g: SimpleGraph[V], t: FiniteSet[V], x: V, y: V) {
    is_independent_in(g, t) and t.contains(x) and t.contains(y) implies not g.adj(x, y)
} by {
    if is_independent_in(g, t) and t.contains(x) and t.contains(y) {
        is_independent_in(g, t) = forall(u: V, w: V) {
            t.contains(u) and t.contains(w) implies not g.adj(u, w)
        }
        forall(u: V, w: V) {
            t.contains(u) and t.contains(w) implies not g.adj(u, w)
        }
        t.contains(x) and t.contains(y) implies not g.adj(x, y)
        not g.adj(x, y)
    }
}

/// A pointwise non-adjacency condition is independence.
theorem is_independent_in_intro[V](g: SimpleGraph[V], t: FiniteSet[V]) {
    (forall(x: V, y: V) {
        t.contains(x) and t.contains(y) implies not g.adj(x, y)
    }) implies is_independent_in(g, t)
} by {
    if forall(x: V, y: V) {
        t.contains(x) and t.contains(y) implies not g.adj(x, y)
    } {
        is_independent_in(g, t) = forall(u: V, w: V) {
            t.contains(u) and t.contains(w) implies not g.adj(u, w)
        }
        is_independent_in(g, t)
    }
}

/// Independence passes to subsets.
theorem is_independent_in_of_subset[V](g: SimpleGraph[V], t: FiniteSet[V], r: FiniteSet[V]) {
    r.subset_eq(t) and is_independent_in(g, t) implies is_independent_in(g, r)
} by {
    if r.subset_eq(t) and is_independent_in(g, t) {
        forall(x: V, y: V) {
            if r.contains(x) and r.contains(y) {
                finite_set_subset_contains(r, t, x)
                finite_set_subset_contains(r, t, y)
                t.contains(x)
                t.contains(y)
                is_independent_in_apply(g, t, x, y)
                not g.adj(x, y)
            }
            r.contains(x) and r.contains(y) implies not g.adj(x, y)
        }
        is_independent_in_intro(g, r)
        is_independent_in(g, r)
    }
}

/// True if every two distinct vertices of `t` are adjacent.
define is_clique_in[V](g: SimpleGraph[V], t: FiniteSet[V]) -> Bool {
    forall(x: V, y: V) {
        t.contains(x) and t.contains(y) and x != y implies g.adj(x, y)
    }
}

/// Two distinct members of a clique are adjacent.
theorem is_clique_in_apply[V](g: SimpleGraph[V], t: FiniteSet[V], x: V, y: V) {
    is_clique_in(g, t) and t.contains(x) and t.contains(y) and x != y implies g.adj(x, y)
} by {
    if is_clique_in(g, t) and t.contains(x) and t.contains(y) and x != y {
        is_clique_in(g, t) = forall(u: V, w: V) {
            t.contains(u) and t.contains(w) and u != w implies g.adj(u, w)
        }
        forall(u: V, w: V) {
            t.contains(u) and t.contains(w) and u != w implies g.adj(u, w)
        }
        t.contains(x) and t.contains(y) and x != y implies g.adj(x, y)
        g.adj(x, y)
    }
}

/// A pointwise adjacency condition is a clique.
theorem is_clique_in_intro[V](g: SimpleGraph[V], t: FiniteSet[V]) {
    (forall(x: V, y: V) {
        t.contains(x) and t.contains(y) and x != y implies g.adj(x, y)
    }) implies is_clique_in(g, t)
} by {
    if forall(x: V, y: V) {
        t.contains(x) and t.contains(y) and x != y implies g.adj(x, y)
    } {
        is_clique_in(g, t) = forall(u: V, w: V) {
            t.contains(u) and t.contains(w) and u != w implies g.adj(u, w)
        }
        is_clique_in(g, t)
    }
}

/// The clique property passes to subsets.
theorem is_clique_in_of_subset[V](g: SimpleGraph[V], t: FiniteSet[V], r: FiniteSet[V]) {
    r.subset_eq(t) and is_clique_in(g, t) implies is_clique_in(g, r)
} by {
    if r.subset_eq(t) and is_clique_in(g, t) {
        forall(x: V, y: V) {
            if r.contains(x) and r.contains(y) and x != y {
                finite_set_subset_contains(r, t, x)
                finite_set_subset_contains(r, t, y)
                t.contains(x)
                t.contains(y)
                is_clique_in_apply(g, t, x, y)
                g.adj(x, y)
            }
            r.contains(x) and r.contains(y) and x != y implies g.adj(x, y)
        }
        is_clique_in_intro(g, r)
        is_clique_in(g, r)
    }
}

/// A set that is both a clique and independent has at most one element.
theorem clique_and_independent_no_two[V](g: SimpleGraph[V], t: FiniteSet[V], x: V, y: V) {
    is_clique_in(g, t) and is_independent_in(g, t) and t.contains(x) and t.contains(y) implies x = y
} by {
    if is_clique_in(g, t) and is_independent_in(g, t) and t.contains(x) and t.contains(y) {
        if x != y {
            is_clique_in_apply(g, t, x, y)
            g.adj(x, y)
            is_independent_in_apply(g, t, x, y)
            not g.adj(x, y)
            false
        }
        x = y
    }
}

/// The complement of an independent set is a vertex cover.
///
/// This is the standard duality: an edge cannot have both endpoints outside the cover,
/// since those endpoints would be two adjacent vertices of the independent set.
theorem vertex_cover_of_independent_complement[V](
    g: SimpleGraph[V], s: FiniteSet[V], t: FiniteSet[V]
) {
    is_independent_in(g, t) implies is_vertex_cover(g, s, fs_difference(s, t))
} by {
    if is_independent_in(g, t) {
        forall(x: V, y: V) {
            if s.contains(x) and s.contains(y) and g.adj(x, y) {
                if not fs_difference(s, t).contains(x) {
                    finite_set_difference_contains_eq(s, t, x)
                    fs_difference(s, t).contains(x) = (s.contains(x) and not t.contains(x))
                    t.contains(x)
                    if not fs_difference(s, t).contains(y) {
                        finite_set_difference_contains_eq(s, t, y)
                        fs_difference(s, t).contains(y) = (s.contains(y) and not t.contains(y))
                        t.contains(y)
                        is_independent_in_apply(g, t, x, y)
                        not g.adj(x, y)
                        false
                    }
                    fs_difference(s, t).contains(y)
                }
                fs_difference(s, t).contains(x) or fs_difference(s, t).contains(y)
            }
            s.contains(x) and s.contains(y) and g.adj(x, y) implies fs_difference(s, t).contains(x) or fs_difference(s, t).contains(y)
        }
        is_vertex_cover_intro(g, s, fs_difference(s, t))
        is_vertex_cover(g, s, fs_difference(s, t))
    }
}
