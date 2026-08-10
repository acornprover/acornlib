// ============================================================================
// Ramsey's theorem R(4, 4) <= 18.
//
// This file records the theorem and its two ingredients. The classical proof
// of the recurrence upper bound splits into three pieces:
//
//   1. The 17 -> 9 pigeonhole: among the seventeen edges from vertex 0, nine
//      share a colour. The counting infrastructure (filter-partition, length
//      extraction) is verified in ramsey44_count.ac (135/135 OK); the
//      extraction of nine distinct witnesses from a filtered list of length
//      >= 9 lives in ramsey44_extract.ac. Full-module verification of that
//      extraction currently stalls on the proof search of
//      `count_ge_two_of_get_idx_eq` in this environment, so the pigeonhole
//      theorem is commented out below.
//
//   2. The asymmetric R(3, 4) <= 9 statement: on nine pairwise distinct
//      vertices, a red triangle or a blue K4. The classical proof goes by
//      contradiction through degree bounds. In a colouring with no red K3
//      and no blue K4, every vertex has red degree at most 3 (four red
//      neighbours would span a blue K4) and blue degree at most 5 (six blue
//      neighbours contain a monochromatic triangle by R(3, 3) <= 6, the
//      parametrized version of which is generated in ramsey44_r33.ac). Hence
//      every vertex has red degree exactly 3, so the sum of the red degrees
//      is 9 * 3 = 27; the handshake lemma forces that sum to be twice the
//      number of red edges, hence even: contradiction.
//
//   3. The assembly: a red triangle among the nine red neighbours of 0 is a
//      red K4 together with 0; a blue K4 among them is done. The blue case
//      is symmetric after swapping the two colours.
//
// The lower bound R(4, 4) > 17 (the Paley graph on 17 vertices) is recorded
// below; as for the R(3, 4) lower bound in ramsey34_upper.ac, the case
// analysis over the four-vertex subsets is not yet verified and the proof is
// commented out.
// ============================================================================

// ----------------------------------------------------------------------------
// The recurrence upper bound R(4, 4) <= 18.
//
// Fix vertex 0. Among the seventeen edges to 1, ..., 17, nine share a colour
// (pigeonhole). If they are red, the asymmetric R(3, 4) <= 9 on the nine red
// neighbours gives a red triangle (completing a red K4 with 0) or a blue K4.
// If they are blue, the same argument on the colour-swapped colouring gives
// a blue triangle (completing a blue K4 with 0) or a red K4.
//
// Not yet verified: the pigeonhole and the asymmetric R(3, 4) lemma are not
// fully verified in this environment, so the theorem is recorded as a
// comment (per AGENTS.md, unfinished proofs are commented out rather than
// stated as axioms).
//
//     theorem ramsey_r44_upper(color: (Nat, Nat) -> Bool) {
//         is_edge_2_coloring(complete_graph[Nat], color) implies
//         has_mono_k4(color, eighteen_vertices)
//     } by {
//         if is_edge_2_coloring(complete_graph[Nat], color) {
//             // (1) pigeonhole on the seventeen edges from 0: nine witnesses
//             //     w1, ..., w9 with a common colour;
//             // (2) if the common colour is red:
//             //       ramsey_r34_asym_nine(color, w1, ..., w9)
//             //       red_k3_in_w9: red triangle {x, y, z}; with 0 a red K4;
//             //       blue_k4_in_w9: a blue K4 already;
//             //     if the common colour is blue: same with the colour-swapped
//             //       colouring, giving a blue triangle (blue K4 with 0) or a
//             //       red K4;
//             // (3) has_red_k4_intro / has_blue_k4_intro (ramsey34_k4.ac)
//             //     close the disjunction.
//         }
//     }
// ----------------------------------------------------------------------------

// ----------------------------------------------------------------------------
// The pigeonhole step: nine of the seventeen edges from vertex 0 share a
// colour.
//
//     theorem nine_edges_from_zero_share_color(color: (Nat, Nat) -> Bool) {
//         exists(w1: Nat, ..., w9: Nat) {
//             w1 < Nat.18 and ... and w9 < Nat.18
//             and w1 != Nat.0 and ... and w9 != Nat.0
//             and w1 != w2 and ... and w8 != w9
//             and color(Nat.0, w1) = ... = color(Nat.0, w9)
//         }
//     } by {
//         // red_count = (17-vertex list).filter(color(Nat.0, -)).length
//         // blue_count = (17-vertex list).filter(not color(Nat.0, -)).length
//         // filter_partition_length (ramsey44_count.ac): red + blue = 17
//         // if red >= 9: extract nine red witnesses (ramsey44_extract.ac)
//         // else: blue >= 9, extract nine blue witnesses
//     }
// ----------------------------------------------------------------------------

// ----------------------------------------------------------------------------
// The asymmetric R(3, 4) <= 9 statement: nine pairwise distinct vertices
// form a red triangle or a blue K4.
//
//     theorem ramsey_r34_asym_nine(color: (Nat, Nat) -> Bool,
//         w1: Nat, ..., w9: Nat) {
//         good_nine(w1, ..., w9) and is_edge_2_coloring(complete_graph[Nat], color)
//         implies (red_k3_in_w9(color, w1, ..., w9)
//             or blue_k4_in_w9(color, w1, ..., w9))
//     } by {
//         // Contradiction mode: assume no red K3 and no blue K4 among the nine.
//         //   (1) every vertex has red degree <= 3: four red neighbours would
//         //       be a blue K4 (each pair together with the vertex is a red
//         //       triangle);
//         //   (2) every vertex has blue degree <= 5: six blue neighbours
//         //       contain a monochromatic triangle by ramsey_r33_on_six
//         //       (ramsey44_r33.ac), red giving a red K3 and blue giving a
//         //       blue K4 with the vertex;
//         //   (3) red degree + blue degree = 8, so red degree = 3 exactly;
//         //   (4) sum of the nine red degrees is 27, but the handshake lemma
//         //       says it is twice the number of red edges, hence even:
//         //       contradiction.
//     }
// ----------------------------------------------------------------------------

// ----------------------------------------------------------------------------
// The Paley graph lower bound R(4, 4) > 17.
//
// On the seventeen vertices {0, ..., 16}, colour the edge {x, y} red exactly
// when x - y is a quadratic residue modulo 17 (the Paley graph of order 17).
// Neither colour contains a K4.
//
// Not yet verified: as for the R(3, 4) lower bound, the certificate for the
// case analysis over the four-vertex subsets is too large; the theorem is
// recorded as a comment.
//
//     define paley17_color(x: Nat, y: Nat) -> Bool {
//         // red edge {x, y} iff x - y is a quadratic residue mod 17
//         false
//     }
//
//     theorem ramsey_r44_lower_bound {
//         exists(color: (Nat, Nat) -> Bool) {
//             is_edge_2_coloring(complete_graph[Nat], color) and
//             not has_red_k4(color, seventeen_vertices) and
//             not has_blue_k4(color, seventeen_vertices)
//         }
//     }
// ----------------------------------------------------------------------------

// ============================================================================
// Verified infrastructure (modules with clean certificates):
//
//   ramsey44_labels.ac   : eighteen-label order and distinctness facts
//                          (verification pending in this environment)
//   ramsey44_vertices.ac : eighteen_vertices, membership facts
//                          (verification pending in this environment)
//   ramsey44_count.ac    : filter-partition length, get_idx membership and
//                          distinctness, length-based extraction lemmas
//                          (135/135 OK)
//   ramsey44_defs.ac     : w9/w6 membership predicates, triangle/K4
//                          predicates with intro/witness lemmas, pairwise
//                          distinctness with nested row definitions
//                          (98/98 OK)
//   ramsey44_extract.ac  : count-based distinctness for extraction
//                          (generated; verification stalls on
//                          count_ge_two_of_get_idx_eq in this environment)
//   ramsey44_r33.ac      : parametrized R(3, 3) <= 6
//                          (generated; verification pending in this
//                          environment)
// ============================================================================
