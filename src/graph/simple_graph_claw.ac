from nat import Nat
from finite_set import FiniteSet, finite_set_subset_contains
from graph.simple_graph import SimpleGraph, complete_graph, complete_graph_adj_iff_ne
from graph.simple_graph_degree import neighborhood, neighborhood_contains_eq

numerals Nat

/// True if `a` has three pairwise non-adjacent neighbors in `s`.
///
/// Named so that claw-freeness below is a one-variable statement. A claw is a four-variable
/// condition written out, which is deeper than the restriction lemmas can handle.
define has_independent_neighbor_triple[V](
    g: SimpleGraph[V], s: FiniteSet[V], a: V
) -> Bool {
    exists(x: V, y: V, z: V) {
        neighborhood(g, s, a).contains(x) and neighborhood(g, s, a).contains(y)
            and neighborhood(g, s, a).contains(z)
            and x != y and x != z and y != z
            and not g.adj(x, y) and not g.adj(x, z) and not g.adj(y, z)
    }
}

/// Three pairwise non-adjacent neighbors witness the condition.
theorem has_independent_neighbor_triple_intro[V](
    g: SimpleGraph[V], s: FiniteSet[V], a: V, x: V, y: V, z: V
) {
    neighborhood(g, s, a).contains(x) and neighborhood(g, s, a).contains(y)
        and neighborhood(g, s, a).contains(z)
        and x != y and x != z and y != z
        and not g.adj(x, y) and not g.adj(x, z) and not g.adj(y, z)
        implies has_independent_neighbor_triple(g, s, a)
} by {
    if neighborhood(g, s, a).contains(x) and neighborhood(g, s, a).contains(y)
        and neighborhood(g, s, a).contains(z)
        and x != y and x != z and y != z
        and not g.adj(x, y) and not g.adj(x, z) and not g.adj(y, z) {
        has_independent_neighbor_triple(g, s, a) = exists(p: V, q: V, r: V) {
            neighborhood(g, s, a).contains(p) and neighborhood(g, s, a).contains(q)
                and neighborhood(g, s, a).contains(r)
                and p != q and p != r and q != r
                and not g.adj(p, q) and not g.adj(p, r) and not g.adj(q, r)
        }
        exists(p: V, q: V, r: V) {
            neighborhood(g, s, a).contains(p) and neighborhood(g, s, a).contains(q)
                and neighborhood(g, s, a).contains(r)
                and p != q and p != r and q != r
                and not g.adj(p, q) and not g.adj(p, r) and not g.adj(q, r)
        }
        has_independent_neighbor_triple(g, s, a)
    }
}

/// Such a triple can be extracted.
theorem has_independent_neighbor_triple_witness[V](
    g: SimpleGraph[V], s: FiniteSet[V], a: V
) {
    has_independent_neighbor_triple(g, s, a) implies exists(x: V, y: V, z: V) {
        neighborhood(g, s, a).contains(x) and neighborhood(g, s, a).contains(y)
            and neighborhood(g, s, a).contains(z)
            and x != y and x != z and y != z
            and not g.adj(x, y) and not g.adj(x, z) and not g.adj(y, z)
    }
} by {
    if has_independent_neighbor_triple(g, s, a) {
        has_independent_neighbor_triple(g, s, a) = exists(p: V, q: V, r: V) {
            neighborhood(g, s, a).contains(p) and neighborhood(g, s, a).contains(q)
                and neighborhood(g, s, a).contains(r)
                and p != q and p != r and q != r
                and not g.adj(p, q) and not g.adj(p, r) and not g.adj(q, r)
        }
        exists(p: V, q: V, r: V) {
            neighborhood(g, s, a).contains(p) and neighborhood(g, s, a).contains(q)
                and neighborhood(g, s, a).contains(r)
                and p != q and p != r and q != r
                and not g.adj(p, q) and not g.adj(p, r) and not g.adj(q, r)
        }
    }
}

/// True if no four vertices of `s` induce a claw.
///
/// A claw is a star on three leaves: a center with three neighbors, no two of which are
/// adjacent. Claw-free graphs are the setting in which several matching and colouring
/// arguments become tractable.
define is_claw_free[V](g: SimpleGraph[V], s: FiniteSet[V]) -> Bool {
    forall(a: V) {
        (s.contains(a) implies not has_independent_neighbor_triple(g, s, a))
    }
}

/// A vertex of a claw-free graph has no three pairwise non-adjacent neighbors.
theorem is_claw_free_apply[V](g: SimpleGraph[V], s: FiniteSet[V], a: V) {
    is_claw_free(g, s) and s.contains(a)
        implies not has_independent_neighbor_triple(g, s, a)
} by {
    if is_claw_free(g, s) and s.contains(a) {
        is_claw_free(g, s) = forall(b: V) {
            (s.contains(b) implies not has_independent_neighbor_triple(g, s, b))
        }
        forall(b: V) {
            (s.contains(b) implies not has_independent_neighbor_triple(g, s, b))
        }
        (s.contains(a) implies not has_independent_neighbor_triple(g, s, a))
        not has_independent_neighbor_triple(g, s, a)
    }
}

/// The pointwise condition is claw-freeness.
theorem is_claw_free_intro[V](g: SimpleGraph[V], s: FiniteSet[V]) {
    (forall(a: V) {
        (s.contains(a) implies not has_independent_neighbor_triple(g, s, a))
    }) implies is_claw_free(g, s)
} by {
    if forall(a: V) {
        (s.contains(a) implies not has_independent_neighbor_triple(g, s, a))
    } {
        is_claw_free(g, s) = forall(b: V) {
            (s.contains(b) implies not has_independent_neighbor_triple(g, s, b))
        }
        is_claw_free(g, s)
    }
}

/// In a claw-free graph, three distinct neighbors of a vertex include an adjacent pair.
///
/// This is the reading in terms of the four vertices themselves, recovered from the
/// neighborhood form.
theorem is_claw_free_no_claw[V](
    g: SimpleGraph[V], s: FiniteSet[V], a: V, x: V, y: V, z: V
) {
    is_claw_free(g, s) and s.contains(a) and s.contains(x) and s.contains(y)
        and s.contains(z) and g.adj(a, x) and g.adj(a, y) and g.adj(a, z)
        and x != y and x != z and y != z
        implies g.adj(x, y) or g.adj(x, z) or g.adj(y, z)
} by {
    if is_claw_free(g, s) and s.contains(a) and s.contains(x) and s.contains(y)
        and s.contains(z) and g.adj(a, x) and g.adj(a, y) and g.adj(a, z)
        and x != y and x != z and y != z {
        if not g.adj(x, y) and not g.adj(x, z) and not g.adj(y, z) {
            neighborhood_contains_eq(g, s, a, x)
            neighborhood(g, s, a).contains(x)
            neighborhood_contains_eq(g, s, a, y)
            neighborhood(g, s, a).contains(y)
            neighborhood_contains_eq(g, s, a, z)
            neighborhood(g, s, a).contains(z)
            has_independent_neighbor_triple_intro(g, s, a, x, y, z)
            has_independent_neighbor_triple(g, s, a)
            is_claw_free_apply(g, s, a)
            not has_independent_neighbor_triple(g, s, a)
            false
        }
        g.adj(x, y) or g.adj(x, z) or g.adj(y, z)
    }
}

/// Claw-freeness passes to subsets of the vertex set.
theorem is_claw_free_of_subset[V](
    g: SimpleGraph[V], s: FiniteSet[V], r: FiniteSet[V]
) {
    r.subset_eq(s) and is_claw_free(g, s) implies is_claw_free(g, r)
} by {
    if r.subset_eq(s) and is_claw_free(g, s) {
        forall(a: V) {
            if r.contains(a) {
                if has_independent_neighbor_triple(g, r, a) {
                    has_independent_neighbor_triple_witness(g, r, a)
                    let (x: V, y: V, z: V) satisfy {
                        neighborhood(g, r, a).contains(x) and neighborhood(g, r, a).contains(y)
                            and neighborhood(g, r, a).contains(z)
                            and x != y and x != z and y != z
                            and not g.adj(x, y) and not g.adj(x, z) and not g.adj(y, z)
                    }
                    neighborhood_contains_eq(g, r, a, x)
                    r.contains(x)
                    finite_set_subset_contains(r, s, x)
                    s.contains(x)
                    neighborhood_contains_eq(g, r, a, y)
                    r.contains(y)
                    finite_set_subset_contains(r, s, y)
                    s.contains(y)
                    neighborhood_contains_eq(g, r, a, z)
                    r.contains(z)
                    finite_set_subset_contains(r, s, z)
                    s.contains(z)
                    finite_set_subset_contains(r, s, a)
                    s.contains(a)
                    neighborhood_contains_eq(g, s, a, x)
                    neighborhood(g, s, a).contains(x)
                    neighborhood_contains_eq(g, s, a, y)
                    neighborhood(g, s, a).contains(y)
                    neighborhood_contains_eq(g, s, a, z)
                    neighborhood(g, s, a).contains(z)
                    has_independent_neighbor_triple_intro(g, s, a, x, y, z)
                    has_independent_neighbor_triple(g, s, a)
                    is_claw_free_apply(g, s, a)
                    not has_independent_neighbor_triple(g, s, a)
                    false
                }
                not has_independent_neighbor_triple(g, r, a)
            }
            (r.contains(a) implies not has_independent_neighbor_triple(g, r, a))
        }
        is_claw_free_intro(g, r)
        is_claw_free(g, r)
    }
}

/// The complete graph is claw-free.
///
/// Distinct vertices are always adjacent there, so no two neighbors of a vertex are
/// independent, let alone three.
theorem complete_graph_is_claw_free[V](s: FiniteSet[V]) {
    is_claw_free(complete_graph[V], s)
} by {
    forall(a: V) {
        if s.contains(a) {
            if has_independent_neighbor_triple(complete_graph[V], s, a) {
                has_independent_neighbor_triple_witness(complete_graph[V], s, a)
                let (x: V, y: V, z: V) satisfy {
                    neighborhood(complete_graph[V], s, a).contains(x)
                        and neighborhood(complete_graph[V], s, a).contains(y)
                        and neighborhood(complete_graph[V], s, a).contains(z)
                        and x != y and x != z and y != z
                        and not complete_graph[V].adj(x, y)
                        and not complete_graph[V].adj(x, z)
                        and not complete_graph[V].adj(y, z)
                }
                complete_graph_adj_iff_ne(x, y)
                complete_graph[V].adj(x, y) = (x != y)
                complete_graph[V].adj(x, y)
                false
            }
            not has_independent_neighbor_triple(complete_graph[V], s, a)
        }
        (s.contains(a) implies not has_independent_neighbor_triple(complete_graph[V], s, a))
    }
    is_claw_free_intro(complete_graph[V], s)
    is_claw_free(complete_graph[V], s)
}
