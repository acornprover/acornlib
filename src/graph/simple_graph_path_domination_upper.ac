from nat import Nat, lte_trans, lt_and_lte, lte_mul_both
from finite_set import FiniteSet, fs_insert, finite_set_subset_contains,
    finite_set_singleton_cardinality_is_one, finite_set_singleton_contains_eq
from data.finite.finite_set_card import fs_card, fs_card_eq_of_cardinality_is, fs_card_empty
from data.finite.finite_set_insert_members import fs_insert_contains_item, fs_insert_contains_of_contains,
    fs_card_insert_le
from data.finite.finite_set_subset_intro import fs_subset_eq_intro
from data.nat.nat_range_set import range_set, range_set_contains, range_set_subset, range_set_suc,
    range_set_lt, range_set_card
from graph.simple_graph_domination import is_dominated, is_dominated_of_contains,
    is_dominated_of_adj, is_dominated_of_subset, is_dominating_set, is_dominating_set_apply,
    is_dominating_set_intro
from graph.simple_graph_domination_number import domination_number, domination_number_attained,
    domination_number_is_least, dominating_size_pred, domination_number_le_ambient
from graph.simple_graph_path import path_graph, path_graph_adj_iff
from graph.simple_graph_path_domination import path_domination_number_lower_bound

numerals Nat

/// A dominating set of a path, with one vertex added, dominates three more.
///
/// The added vertex is `n + 1`, the middle of the three new ones, so it dominates all of them
/// at once. The path relation is the same for every length, so the old vertices stay dominated
/// by the old set.
theorem path_insert_dominates(n: Nat, d: FiniteSet[Nat]) {
    is_dominating_set(path_graph, range_set(n), d)
        implies is_dominating_set(path_graph, range_set(n.suc.suc.suc),
            fs_insert(d, n.suc))
} by {
    if is_dominating_set(path_graph, range_set(n), d) {
        forall(x: Nat) {
            if d.contains(x) {
                fs_insert_contains_of_contains(d, n.suc, x)
                fs_insert(d, n.suc).contains(x)
            }
            (d.contains(x) implies fs_insert(d, n.suc).contains(x))
        }
        fs_subset_eq_intro(d, fs_insert(d, n.suc))
        d.subset_eq(fs_insert(d, n.suc))
        fs_insert_contains_item(d, n.suc)
        fs_insert(d, n.suc).contains(n.suc)
        path_graph_adj_iff(n.suc, n)
        (path_graph.adj(n.suc, n) = (n.suc.suc = n or n.suc = n.suc))
        path_graph.adj(n.suc, n)
        path_graph_adj_iff(n.suc, n.suc.suc)
        (path_graph.adj(n.suc, n.suc.suc)
            = (n.suc.suc = n.suc.suc or n.suc.suc.suc = n.suc))
        path_graph.adj(n.suc, n.suc.suc)
        forall(v: Nat) {
            if range_set(n.suc.suc.suc).contains(v) {
                range_set_suc(n.suc.suc, v)
                (range_set(n.suc.suc.suc).contains(v)
                    = (range_set(n.suc.suc).contains(v) or v = n.suc.suc))
                if v = n.suc.suc {
                    is_dominated_of_adj(path_graph, fs_insert(d, n.suc), v, n.suc)
                    is_dominated(path_graph, fs_insert(d, n.suc), v)
                }
                if range_set(n.suc.suc).contains(v) {
                    range_set_suc(n.suc, v)
                    (range_set(n.suc.suc).contains(v)
                        = (range_set(n.suc).contains(v) or v = n.suc))
                    if v = n.suc {
                        is_dominated_of_contains(path_graph, fs_insert(d, n.suc), v)
                        is_dominated(path_graph, fs_insert(d, n.suc), v)
                    }
                    if range_set(n.suc).contains(v) {
                        range_set_suc(n, v)
                        (range_set(n.suc).contains(v)
                            = (range_set(n).contains(v) or v = n))
                        if v = n {
                            is_dominated_of_adj(path_graph, fs_insert(d, n.suc), v, n.suc)
                            is_dominated(path_graph, fs_insert(d, n.suc), v)
                        }
                        if range_set(n).contains(v) {
                            is_dominating_set_apply(path_graph, range_set(n), d, v)
                            is_dominated(path_graph, d, v)
                            is_dominated_of_subset(path_graph, d, fs_insert(d, n.suc), v)
                            is_dominated(path_graph, fs_insert(d, n.suc), v)
                        }
                        is_dominated(path_graph, fs_insert(d, n.suc), v)
                    }
                    is_dominated(path_graph, fs_insert(d, n.suc), v)
                }
                is_dominated(path_graph, fs_insert(d, n.suc), v)
            }
            (range_set(n.suc.suc.suc).contains(v)
                implies is_dominated(path_graph, fs_insert(d, n.suc), v))
        }
        is_dominating_set_intro(path_graph, range_set(n.suc.suc.suc),
            fs_insert(d, n.suc))
        is_dominating_set(path_graph, range_set(n.suc.suc.suc), fs_insert(d, n.suc))
    }
}

/// Extending a path by three vertices costs at most one more dominating vertex.
theorem path_domination_number_step(n: Nat) {
    domination_number(path_graph, range_set(n.suc.suc.suc))
        <= domination_number(path_graph, range_set(n)) + Nat.1
} by {
    domination_number_attained(path_graph, range_set(n))
    dominating_size_pred(path_graph, range_set(n))(
        domination_number(path_graph, range_set(n)))
    (dominating_size_pred(path_graph, range_set(n))(
        domination_number(path_graph, range_set(n))) = exists(d: FiniteSet[Nat]) {
        d.subset_eq(range_set(n)) and is_dominating_set(path_graph, range_set(n), d)
            and fs_card(d) = domination_number(path_graph, range_set(n))
    })
    exists(d: FiniteSet[Nat]) {
        d.subset_eq(range_set(n)) and is_dominating_set(path_graph, range_set(n), d)
            and fs_card(d) = domination_number(path_graph, range_set(n))
    }
    let (e: FiniteSet[Nat]) satisfy {
        e.subset_eq(range_set(n)) and is_dominating_set(path_graph, range_set(n), e)
            and fs_card(e) = domination_number(path_graph, range_set(n))
    }
    n <= n.suc
    n.suc <= n.suc.suc
    n.suc.suc <= n.suc.suc.suc
    lte_trans(n, n.suc, n.suc.suc)
    n <= n.suc.suc
    lte_trans(n, n.suc.suc, n.suc.suc.suc)
    n <= n.suc.suc.suc
    range_set_subset(n, n.suc.suc.suc)
    range_set(n).subset_eq(range_set(n.suc.suc.suc))
    n.suc < n.suc.suc.suc
    range_set_contains(n.suc.suc.suc, n.suc)
    range_set(n.suc.suc.suc).contains(n.suc)
    forall(x: Nat) {
        if fs_insert(e, n.suc).contains(x) {
            if x = n.suc {
                range_set(n.suc.suc.suc).contains(x)
            }
            if x != n.suc {
                e.contains(x)
                finite_set_subset_contains(e, range_set(n), x)
                range_set(n).contains(x)
                finite_set_subset_contains(range_set(n), range_set(n.suc.suc.suc), x)
                range_set(n.suc.suc.suc).contains(x)
            }
            range_set(n.suc.suc.suc).contains(x)
        }
        (fs_insert(e, n.suc).contains(x) implies range_set(n.suc.suc.suc).contains(x))
    }
    fs_subset_eq_intro(fs_insert(e, n.suc), range_set(n.suc.suc.suc))
    fs_insert(e, n.suc).subset_eq(range_set(n.suc.suc.suc))
    path_insert_dominates(n, e)
    is_dominating_set(path_graph, range_set(n.suc.suc.suc), fs_insert(e, n.suc))
    domination_number_is_least(path_graph, range_set(n.suc.suc.suc), fs_insert(e, n.suc))
    domination_number(path_graph, range_set(n.suc.suc.suc)) <= fs_card(fs_insert(e, n.suc))
    fs_card_insert_le(e, n.suc)
    fs_card(fs_insert(e, n.suc)) <= fs_card(e) + Nat.1
    lte_trans(domination_number(path_graph, range_set(n.suc.suc.suc)),
        fs_card(fs_insert(e, n.suc)), fs_card(e) + Nat.1)
    (domination_number(path_graph, range_set(n.suc.suc.suc))
        <= domination_number(path_graph, range_set(n)) + Nat.1)
}

/// The two-vertex path is dominated by a single vertex.
///
/// The one base case the recursion cannot get from the ambient bound: `P_2` has two vertices
/// but needs only one, since its two vertices are adjacent.
theorem path_two_domination_number {
    domination_number(path_graph, range_set(Nat.2)) <= Nat.1
} by {
    Nat.0 < Nat.2
    range_set_contains(Nat.2, Nat.0)
    range_set(Nat.2).contains(Nat.0)
    fs_insert_contains_item(FiniteSet.empty[Nat], Nat.0)
    fs_insert(FiniteSet.empty[Nat], Nat.0).contains(Nat.0)
    forall(x: Nat) {
        if fs_insert(FiniteSet.empty[Nat], Nat.0).contains(x) {
            (fs_insert(FiniteSet.empty[Nat], Nat.0).contains(x) = (x = Nat.0))
            x = Nat.0
            range_set(Nat.2).contains(x)
        }
        (fs_insert(FiniteSet.empty[Nat], Nat.0).contains(x)
            implies range_set(Nat.2).contains(x))
    }
    fs_subset_eq_intro(fs_insert(FiniteSet.empty[Nat], Nat.0), range_set(Nat.2))
    fs_insert(FiniteSet.empty[Nat], Nat.0).subset_eq(range_set(Nat.2))
    path_graph_adj_iff(Nat.0, Nat.1)
    (Nat.0.suc = Nat.1)
    (path_graph.adj(Nat.0, Nat.1) = (Nat.0.suc = Nat.1 or Nat.1.suc = Nat.0))
    path_graph.adj(Nat.0, Nat.1)
    forall(v: Nat) {
        if range_set(Nat.2).contains(v) {
            (Nat.2 = Nat.1.suc)
            range_set_suc(Nat.1, v)
            (range_set(Nat.1.suc).contains(v)
                = (range_set(Nat.1).contains(v) or v = Nat.1))
            if v = Nat.1 {
                is_dominated_of_adj(path_graph,
                    fs_insert(FiniteSet.empty[Nat], Nat.0), v, Nat.0)
                is_dominated(path_graph, fs_insert(FiniteSet.empty[Nat], Nat.0), v)
            }
            if range_set(Nat.1).contains(v) {
                (Nat.1 = Nat.0.suc)
                range_set_suc(Nat.0, v)
                (range_set(Nat.0.suc).contains(v)
                    = (range_set(Nat.0).contains(v) or v = Nat.0))
                range_set_lt(Nat.0, v)
                not range_set(Nat.0).contains(v)
                v = Nat.0
                is_dominated_of_contains(path_graph,
                    fs_insert(FiniteSet.empty[Nat], Nat.0), v)
                is_dominated(path_graph, fs_insert(FiniteSet.empty[Nat], Nat.0), v)
            }
            is_dominated(path_graph, fs_insert(FiniteSet.empty[Nat], Nat.0), v)
        }
        (range_set(Nat.2).contains(v)
            implies is_dominated(path_graph, fs_insert(FiniteSet.empty[Nat], Nat.0), v))
    }
    is_dominating_set_intro(path_graph, range_set(Nat.2),
        fs_insert(FiniteSet.empty[Nat], Nat.0))
    is_dominating_set(path_graph, range_set(Nat.2),
        fs_insert(FiniteSet.empty[Nat], Nat.0))
    domination_number_is_least(path_graph, range_set(Nat.2),
        fs_insert(FiniteSet.empty[Nat], Nat.0))
    (domination_number(path_graph, range_set(Nat.2))
        <= fs_card(fs_insert(FiniteSet.empty[Nat], Nat.0)))
    finite_set_singleton_cardinality_is_one[Nat](Nat.0)
    fs_insert(FiniteSet.empty[Nat], Nat.0).cardinality_is(Nat.1)
    fs_card_eq_of_cardinality_is(fs_insert(FiniteSet.empty[Nat], Nat.0), Nat.1)
    fs_card(fs_insert(FiniteSet.empty[Nat], Nat.0)) = Nat.1
    domination_number(path_graph, range_set(Nat.2)) <= Nat.1
}

/// The domination number of a path is at most a third of its length, rounded up.
///
/// Stated as `3 * gamma <= n + 2` so that no ceiling appears. Together with the lower bound
/// `n <= 3 * gamma` this pins the domination number exactly, since the two together leave only
/// one value for it.
///
/// The recursion has step three, so the induction carries three consecutive lengths at once:
/// the three base cases are `P_0`, `P_1`, and `P_2`, and each step slides the window along by
/// one while proving the length three further out.
theorem path_domination_number_upper_bound(n: Nat) {
    Nat.3 * domination_number(path_graph, range_set(n)) <= n + Nat.2
} by {
    define q(k: Nat) -> Bool {
        Nat.3 * domination_number(path_graph, range_set(k)) <= k + Nat.2
    }
    forall(m: Nat) {
        if q(m) {
            (Nat.3 * domination_number(path_graph, range_set(m)) <= m + Nat.2)
            path_domination_number_step(m)
            (domination_number(path_graph, range_set(m.suc.suc.suc))
                <= domination_number(path_graph, range_set(m)) + Nat.1)
            lte_mul_both(Nat.3, domination_number(path_graph, range_set(m.suc.suc.suc)),
                domination_number(path_graph, range_set(m)) + Nat.1)
            (Nat.3 * domination_number(path_graph, range_set(m.suc.suc.suc))
                <= Nat.3 * (domination_number(path_graph, range_set(m)) + Nat.1))
            (Nat.3 * (domination_number(path_graph, range_set(m)) + Nat.1)
                = Nat.3 * domination_number(path_graph, range_set(m)) + Nat.3)
            (Nat.3 * domination_number(path_graph, range_set(m)) + Nat.3
                <= (m + Nat.2) + Nat.3)
            lte_trans(Nat.3 * domination_number(path_graph, range_set(m.suc.suc.suc)),
                Nat.3 * domination_number(path_graph, range_set(m)) + Nat.3,
                (m + Nat.2) + Nat.3)
            (Nat.3 * domination_number(path_graph, range_set(m.suc.suc.suc))
                <= (m + Nat.2) + Nat.3)
            ((m + Nat.2) + Nat.3 = m.suc.suc.suc + Nat.2)
            (Nat.3 * domination_number(path_graph, range_set(m.suc.suc.suc))
                <= m.suc.suc.suc + Nat.2)
            q(m.suc.suc.suc)
        }
        (q(m) implies q(m.suc.suc.suc))
    }
    domination_number_le_ambient(path_graph, range_set(Nat.0))
    range_set_card(Nat.0)
    fs_card(range_set(Nat.0)) = Nat.0
    domination_number(path_graph, range_set(Nat.0)) <= Nat.0
    domination_number(path_graph, range_set(Nat.0)) = Nat.0
    (Nat.3 * Nat.0 = Nat.0)
    (Nat.0 <= Nat.0 + Nat.2)
    q(Nat.0)
    domination_number_le_ambient(path_graph, range_set(Nat.1))
    range_set_card(Nat.1)
    fs_card(range_set(Nat.1)) = Nat.1
    domination_number(path_graph, range_set(Nat.1)) <= Nat.1
    lte_mul_both(Nat.3, domination_number(path_graph, range_set(Nat.1)), Nat.1)
    (Nat.3 * domination_number(path_graph, range_set(Nat.1)) <= Nat.3 * Nat.1)
    (Nat.3 * Nat.1 = Nat.1 + Nat.2)
    q(Nat.1)
    path_two_domination_number
    domination_number(path_graph, range_set(Nat.2)) <= Nat.1
    lte_mul_both(Nat.3, domination_number(path_graph, range_set(Nat.2)), Nat.1)
    (Nat.3 * domination_number(path_graph, range_set(Nat.2)) <= Nat.3 * Nat.1)
    (Nat.3 * Nat.1 = Nat.3)
    (Nat.2 + Nat.2 = Nat.4)
    (Nat.3.suc = Nat.4)
    Nat.3 < Nat.3.suc
    Nat.3 <= Nat.4
    (Nat.3 * Nat.1 <= Nat.2 + Nat.2)
    lte_trans(Nat.3 * domination_number(path_graph, range_set(Nat.2)),
        Nat.3 * Nat.1, Nat.2 + Nat.2)
    q(Nat.2)
    define w(m: Nat) -> Bool {
        q(m) and q(m.suc) and q(m.suc.suc)
    }
    (Nat.0.suc = Nat.1)
    (Nat.1.suc = Nat.2)
    w(Nat.0)
    forall(m: Nat) {
        if w(m) {
            q(m)
            q(m.suc)
            q(m.suc.suc)
            (q(m) implies q(m.suc.suc.suc))
            q(m.suc.suc.suc)
            w(m.suc)
        }
        (w(m) implies w(m.suc))
    }
    w(Nat.0) and forall(m: Nat) {
        w(m) implies w(m.suc)
    }
    Nat.induction(w)
    w(n)
    q(n)
    (Nat.3 * domination_number(path_graph, range_set(n)) <= n + Nat.2)
}

/// The domination number of a path, pinned between two bounds.
///
/// `3 * gamma` lies in `{n, n + 1, n + 2}`, which leaves exactly one value for `gamma`: it is
/// the length divided by three and rounded up. Stated as the pair of bounds rather than through
/// a ceiling, since the pair is what a counting argument supplies and what one consumes.
theorem path_domination_number_bounds(n: Nat) {
    n <= Nat.3 * domination_number(path_graph, range_set(n))
        and Nat.3 * domination_number(path_graph, range_set(n)) <= n + Nat.2
} by {
    path_domination_number_lower_bound(n)
    n <= Nat.3 * domination_number(path_graph, range_set(n))
    path_domination_number_upper_bound(n)
    (Nat.3 * domination_number(path_graph, range_set(n)) <= n + Nat.2)
}
