from nat import Nat
from finite_set import FiniteSet, finite_set_subset_contains
from graph.simple_graph import SimpleGraph
from graph.simple_graph_degree import degree, neighborhood, neighborhood_contains_of_adj
from data.finite.finite_set_card import fs_card
from data.finite.finite_set_card_ops import fs_card_positive_of_contains

numerals Nat

/// True if `part` two-colours the vertices of `s` so that no edge joins two vertices
/// of the same side.
///
/// The colouring is a predicate rather than a set, so the two sides are `part` and its
/// complement and no disjointness hypothesis is needed.
define is_bipartition[V](g: SimpleGraph[V], s: FiniteSet[V], part: V -> Bool) -> Bool {
    forall(x: V, y: V) {
        s.contains(x) and s.contains(y) and g.adj(x, y) implies part(x) != part(y)
    }
}

/// Adjacent vertices of a bipartition lie on opposite sides.
theorem is_bipartition_apply[V](
    g: SimpleGraph[V], s: FiniteSet[V], part: V -> Bool, x: V, y: V
) {
    is_bipartition(g, s, part) and s.contains(x) and s.contains(y) and g.adj(x, y)
        implies part(x) != part(y)
} by {
    if is_bipartition(g, s, part) and s.contains(x) and s.contains(y) and g.adj(x, y) {
        is_bipartition(g, s, part) = forall(u: V, w: V) {
            s.contains(u) and s.contains(w) and g.adj(u, w) implies part(u) != part(w)
        }
        forall(u: V, w: V) {
            s.contains(u) and s.contains(w) and g.adj(u, w) implies part(u) != part(w)
        }
        s.contains(x) and s.contains(y) and g.adj(x, y) implies part(x) != part(y)
        part(x) != part(y)
    }
}

/// A pointwise opposite-sides condition is a bipartition.
theorem is_bipartition_intro[V](g: SimpleGraph[V], s: FiniteSet[V], part: V -> Bool) {
    (forall(x: V, y: V) {
        s.contains(x) and s.contains(y) and g.adj(x, y) implies part(x) != part(y)
    }) implies is_bipartition(g, s, part)
} by {
    if forall(x: V, y: V) {
        s.contains(x) and s.contains(y) and g.adj(x, y) implies part(x) != part(y)
    } {
        is_bipartition(g, s, part) = forall(u: V, w: V) {
            s.contains(u) and s.contains(w) and g.adj(u, w) implies part(u) != part(w)
        }
        is_bipartition(g, s, part)
    }
}

/// Two vertices on the same side of a bipartition are not adjacent.
theorem is_bipartition_same_side_not_adj[V](
    g: SimpleGraph[V], s: FiniteSet[V], part: V -> Bool, x: V, y: V
) {
    is_bipartition(g, s, part) and s.contains(x) and s.contains(y) and part(x) = part(y)
        implies not g.adj(x, y)
} by {
    if is_bipartition(g, s, part) and s.contains(x) and s.contains(y) and part(x) = part(y) {
        if g.adj(x, y) {
            is_bipartition_apply(g, s, part, x, y)
            part(x) != part(y)
            false
        }
        not g.adj(x, y)
    }
}

/// A bipartition of a set restricts to any subset.
theorem is_bipartition_of_subset[V](
    g: SimpleGraph[V], s: FiniteSet[V], t: FiniteSet[V], part: V -> Bool
) {
    t.subset_eq(s) and is_bipartition(g, s, part) implies is_bipartition(g, t, part)
} by {
    if t.subset_eq(s) and is_bipartition(g, s, part) {
        forall(x: V, y: V) {
            if t.contains(x) and t.contains(y) and g.adj(x, y) {
                finite_set_subset_contains(t, s, x)
                finite_set_subset_contains(t, s, y)
                s.contains(x)
                s.contains(y)
                is_bipartition_apply(g, s, part, x, y)
                part(x) != part(y)
            }
            t.contains(x) and t.contains(y) and g.adj(x, y) implies part(x) != part(y)
        }
        is_bipartition_intro(g, t, part)
        is_bipartition(g, t, part)
    }
}

/// True if some two-colouring of `s` makes `g` bipartite.
define is_bipartite[V](g: SimpleGraph[V], s: FiniteSet[V]) -> Bool {
    exists(part: V -> Bool) {
        is_bipartition(g, s, part)
    }
}

/// A bipartition witnesses bipartiteness.
theorem is_bipartite_intro[V](g: SimpleGraph[V], s: FiniteSet[V], part: V -> Bool) {
    is_bipartition(g, s, part) implies is_bipartite(g, s)
} by {
    if is_bipartition(g, s, part) {
        is_bipartite(g, s) = exists(other: V -> Bool) {
            is_bipartition(g, s, other)
        }
        exists(other: V -> Bool) {
            is_bipartition(g, s, other)
        }
        is_bipartite(g, s)
    }
}

/// A bipartite graph has a bipartition.
theorem is_bipartite_witness[V](g: SimpleGraph[V], s: FiniteSet[V]) {
    is_bipartite(g, s) implies exists(part: V -> Bool) { is_bipartition(g, s, part) }
} by {
    if is_bipartite(g, s) {
        is_bipartite(g, s) = exists(other: V -> Bool) {
            is_bipartition(g, s, other)
        }
        exists(other: V -> Bool) {
            is_bipartition(g, s, other)
        }
    }
}

/// Bipartiteness passes to subsets of the vertex set.
theorem is_bipartite_of_subset[V](g: SimpleGraph[V], s: FiniteSet[V], t: FiniteSet[V]) {
    t.subset_eq(s) and is_bipartite(g, s) implies is_bipartite(g, t)
} by {
    if t.subset_eq(s) and is_bipartite(g, s) {
        is_bipartite_witness(g, s)
        let (part: V -> Bool) satisfy {
            is_bipartition(g, s, part)
        }
        is_bipartition_of_subset(g, s, t, part)
        is_bipartition(g, t, part)
        is_bipartite_intro(g, t, part)
        is_bipartite(g, t)
    }
}

/// True if every vertex of `s` has at least one neighbor in `s`.
define has_no_isolated_vertices[V](g: SimpleGraph[V], s: FiniteSet[V]) -> Bool {
    forall(v: V) {
        s.contains(v) implies Nat.0 < degree(g, s, v)
    }
}

/// A vertex of a graph without isolated vertices has positive degree.
theorem has_no_isolated_vertices_apply[V](g: SimpleGraph[V], s: FiniteSet[V], v: V) {
    has_no_isolated_vertices(g, s) and s.contains(v) implies Nat.0 < degree(g, s, v)
} by {
    if has_no_isolated_vertices(g, s) and s.contains(v) {
        has_no_isolated_vertices(g, s) = forall(u: V) {
            s.contains(u) implies Nat.0 < degree(g, s, u)
        }
        forall(u: V) {
            s.contains(u) implies Nat.0 < degree(g, s, u)
        }
        s.contains(v) implies Nat.0 < degree(g, s, v)
        Nat.0 < degree(g, s, v)
    }
}

/// Pointwise positive degree means no isolated vertices.
theorem has_no_isolated_vertices_intro[V](g: SimpleGraph[V], s: FiniteSet[V]) {
    (forall(v: V) { s.contains(v) implies Nat.0 < degree(g, s, v) })
        implies has_no_isolated_vertices(g, s)
} by {
    if forall(v: V) { s.contains(v) implies Nat.0 < degree(g, s, v) } {
        has_no_isolated_vertices(g, s) = forall(u: V) {
            s.contains(u) implies Nat.0 < degree(g, s, u)
        }
        has_no_isolated_vertices(g, s)
    }
}

/// A vertex with a neighbor in the ambient set has positive degree.
theorem degree_positive_of_adj[V](g: SimpleGraph[V], s: FiniteSet[V], v: V, w: V) {
    s.contains(w) and g.adj(v, w) implies Nat.0 < degree(g, s, v)
} by {
    if s.contains(w) and g.adj(v, w) {
        neighborhood_contains_of_adj(g, s, v, w)
        neighborhood(g, s, v).contains(w)
        fs_card_positive_of_contains(neighborhood(g, s, v), w)
        Nat.0 < fs_card(neighborhood(g, s, v))
        Nat.0 < degree(g, s, v)
    }
}
