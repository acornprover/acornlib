from nat import Nat, add_one_right, lte_and_lt
from finite_set import FiniteSet, fs_remove, finite_set_subset_antisymm
from data.finite.finite_set_membership import fs_remove_contains_eq, fs_remove_subset
from data.finite.finite_set_card import fs_card
from data.finite.finite_set_card_ops import fs_card_remove_of_contains
from data.finite.finite_set_subset_intro import fs_subset_eq_intro
from graph.simple_graph import SimpleGraph, complete_graph, complete_graph_adj_iff_ne
from graph.simple_graph_zero_forcing_rule import is_white_neighbor, can_force, can_force_intro,
    derived_set, derived_set_contains_eq, derived_set_subset, is_forceable, is_forceable_intro
from graph.simple_graph_zero_forcing_closure import forcing_iterate, forcing_iterate_zero,
    forcing_iterate_suc, is_zero_forcing_set, is_zero_forcing_set_intro
from graph.simple_graph_zero_forcing_number import zero_forcing_number, zero_forcing_number_is_least

numerals Nat

/// In the complete graph, every vertex but one forces the remaining one.
///
/// The complement of a single vertex leaves exactly one white vertex, and in the complete
/// graph every blue vertex is adjacent to it, so each of them sees it as its only white
/// neighbor.
theorem complete_graph_can_force_last[V](
    s: FiniteSet[V], v: V, u: V
) {
    s.contains(v) and fs_remove(s, v).contains(u)
        implies can_force(complete_graph[V], s, fs_remove(s, v), u, v)
} by {
    if s.contains(v) and fs_remove(s, v).contains(u) {
        fs_remove_contains_eq(s, v, u)
        s.contains(u) and u != v
        u != v
        fs_remove_contains_eq(s, v, v)
        not fs_remove(s, v).contains(v)
        complete_graph_adj_iff_ne(u, v)
        complete_graph[V].adj(u, v) = (u != v)
        complete_graph[V].adj(u, v)
        (is_white_neighbor(complete_graph[V], s, fs_remove(s, v), u, v)
            = (s.contains(v) and not fs_remove(s, v).contains(v)
                and complete_graph[V].adj(u, v)))
        is_white_neighbor(complete_graph[V], s, fs_remove(s, v), u, v)
        forall(w: V) {
            if is_white_neighbor(complete_graph[V], s, fs_remove(s, v), u, w) {
                (is_white_neighbor(complete_graph[V], s, fs_remove(s, v), u, w)
                    = (s.contains(w) and not fs_remove(s, v).contains(w)
                        and complete_graph[V].adj(u, w)))
                s.contains(w)
                not fs_remove(s, v).contains(w)
                fs_remove_contains_eq(s, v, w)
                not (s.contains(w) and w != v)
                w = v
            }
            (is_white_neighbor(complete_graph[V], s, fs_remove(s, v), u, w) implies w = v)
        }
        can_force_intro(complete_graph[V], s, fs_remove(s, v), u, v)
        can_force(complete_graph[V], s, fs_remove(s, v), u, v)
    }
}

/// One round of forcing from the complement of a vertex colours the whole set.
theorem complete_graph_derived_set_all[V](s: FiniteSet[V], v: V, u: V) {
    s.contains(v) and fs_remove(s, v).contains(u)
        implies derived_set(complete_graph[V], s, fs_remove(s, v)) = s
} by {
    if s.contains(v) and fs_remove(s, v).contains(u) {
        complete_graph_can_force_last(s, v, u)
        can_force(complete_graph[V], s, fs_remove(s, v), u, v)
        is_forceable_intro(complete_graph[V], s, fs_remove(s, v), u, v)
        is_forceable(complete_graph[V], s, fs_remove(s, v), v)
        forall(x: V) {
            if s.contains(x) {
                if x = v {
                    derived_set_contains_eq(complete_graph[V], s, fs_remove(s, v), x)
                    derived_set(complete_graph[V], s, fs_remove(s, v)).contains(x)
                }
                if x != v {
                    fs_remove_contains_eq(s, v, x)
                    fs_remove(s, v).contains(x)
                    derived_set_contains_eq(complete_graph[V], s, fs_remove(s, v), x)
                    derived_set(complete_graph[V], s, fs_remove(s, v)).contains(x)
                }
                derived_set(complete_graph[V], s, fs_remove(s, v)).contains(x)
            }
            (s.contains(x) implies derived_set(complete_graph[V], s, fs_remove(s, v)).contains(x))
        }
        fs_subset_eq_intro(s, derived_set(complete_graph[V], s, fs_remove(s, v)))
        s.subset_eq(derived_set(complete_graph[V], s, fs_remove(s, v)))
        derived_set_subset(complete_graph[V], s, fs_remove(s, v))
        derived_set(complete_graph[V], s, fs_remove(s, v)).subset_eq(s)
        finite_set_subset_antisymm(derived_set(complete_graph[V], s, fs_remove(s, v)), s)
        derived_set(complete_graph[V], s, fs_remove(s, v)) = s
    }
}

/// All but one vertex is a zero forcing set of the complete graph.
theorem complete_graph_remove_is_zero_forcing[V](s: FiniteSet[V], v: V, u: V) {
    s.contains(v) and fs_remove(s, v).contains(u)
        implies is_zero_forcing_set(complete_graph[V], s, fs_remove(s, v))
} by {
    if s.contains(v) and fs_remove(s, v).contains(u) {
        fs_remove_subset(s, v)
        fs_remove(s, v).subset_eq(s)
        forcing_iterate_zero(complete_graph[V], s, fs_remove(s, v))
        forcing_iterate(complete_graph[V], s, fs_remove(s, v), Nat.0) = fs_remove(s, v)
        forcing_iterate_suc(complete_graph[V], s, fs_remove(s, v), Nat.0)
        (forcing_iterate(complete_graph[V], s, fs_remove(s, v), Nat.0.suc)
            = derived_set(complete_graph[V], s,
                forcing_iterate(complete_graph[V], s, fs_remove(s, v), Nat.0)))
        (forcing_iterate(complete_graph[V], s, fs_remove(s, v), Nat.0.suc)
            = derived_set(complete_graph[V], s, fs_remove(s, v)))
        complete_graph_derived_set_all(s, v, u)
        derived_set(complete_graph[V], s, fs_remove(s, v)) = s
        forcing_iterate(complete_graph[V], s, fs_remove(s, v), Nat.0.suc) = s
        is_zero_forcing_set_intro(complete_graph[V], s, fs_remove(s, v), Nat.0.suc)
        is_zero_forcing_set(complete_graph[V], s, fs_remove(s, v))
    }
}

/// The zero forcing number of the complete graph is less than the number of vertices.
///
/// One round suffices from all but one vertex, so the invariant is at most one below the
/// vertex count. Two distinct vertices are needed: with fewer there is no blue vertex to do
/// the forcing.
theorem complete_graph_zero_forcing_number_lt[V](s: FiniteSet[V], v: V, u: V) {
    s.contains(v) and s.contains(u) and u != v
        implies zero_forcing_number(complete_graph[V], s) < fs_card(s)
} by {
    if s.contains(v) and s.contains(u) and u != v {
        fs_remove_contains_eq(s, v, u)
        fs_remove(s, v).contains(u)
        complete_graph_remove_is_zero_forcing(s, v, u)
        is_zero_forcing_set(complete_graph[V], s, fs_remove(s, v))
        zero_forcing_number_is_least(complete_graph[V], s, fs_remove(s, v))
        zero_forcing_number(complete_graph[V], s) <= fs_card(fs_remove(s, v))
        fs_card_remove_of_contains(s, v)
        fs_card(s) = fs_card(fs_remove(s, v)) + Nat.1
        add_one_right(fs_card(fs_remove(s, v)))
        fs_card(fs_remove(s, v)) + Nat.1 = fs_card(fs_remove(s, v)).suc
        fs_card(s) = fs_card(fs_remove(s, v)).suc
        fs_card(fs_remove(s, v)) < fs_card(fs_remove(s, v)).suc
        fs_card(fs_remove(s, v)) < fs_card(s)
        lte_and_lt(zero_forcing_number(complete_graph[V], s), fs_card(fs_remove(s, v)),
            fs_card(s))
        zero_forcing_number(complete_graph[V], s) < fs_card(s)
    }
}
