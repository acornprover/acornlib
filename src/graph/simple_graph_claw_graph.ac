from nat import Nat
from data.fin.fin import Fin
from data.basic.relation_basic import is_symmetric, is_irreflexive
from graph.simple_graph import SimpleGraph

numerals Nat

/// Adjacency on the four-vertex claw: the center is `0` and the leaves are `1`, `2`, `3`.
///
/// Written as a disjunction rather than as an inequality of booleans, since proof search does
/// not case split on a boolean and the inequality form leaves every step to it.
define claw4_adj(x: Fin[Nat.4], y: Fin[Nat.4]) -> Bool {
    (x.value = Nat.0 and y.value != Nat.0) or (y.value = Nat.0 and x.value != Nat.0)
}

/// Claw adjacency is symmetric.
theorem claw4_adj_is_symmetric {
    is_symmetric(claw4_adj)
} by {
    forall(x: Fin[Nat.4], y: Fin[Nat.4]) {
        if claw4_adj(x, y) {
            (claw4_adj(x, y) = ((x.value = Nat.0 and y.value != Nat.0)
                or (y.value = Nat.0 and x.value != Nat.0)))
            (claw4_adj(y, x) = ((y.value = Nat.0 and x.value != Nat.0)
                or (x.value = Nat.0 and y.value != Nat.0)))
            claw4_adj(y, x)
        }
        (claw4_adj(x, y) implies claw4_adj(y, x))
    }
    (is_symmetric(claw4_adj) = forall(x: Fin[Nat.4], y: Fin[Nat.4]) {
        claw4_adj(x, y) implies claw4_adj(y, x)
    })
    is_symmetric(claw4_adj)
}

/// No vertex of the claw is adjacent to itself.
theorem claw4_adj_is_irreflexive {
    is_irreflexive(claw4_adj)
} by {
    forall(x: Fin[Nat.4]) {
        if claw4_adj(x, x) {
            (claw4_adj(x, x) = ((x.value = Nat.0 and x.value != Nat.0)
                or (x.value = Nat.0 and x.value != Nat.0)))
            (x.value = Nat.0 and x.value != Nat.0)
            false
        }
        not claw4_adj(x, x)
    }
    (is_irreflexive(claw4_adj) = forall(x: Fin[Nat.4]) { not claw4_adj(x, x) })
    is_irreflexive(claw4_adj)
}

/// `claw4_adj` satisfies the simple-graph constraint.
theorem claw4_adj_constraint {
    SimpleGraph.constraint(claw4_adj)
} by {
    claw4_adj_is_symmetric
    claw4_adj_is_irreflexive
    (SimpleGraph.constraint(claw4_adj)
        = (is_symmetric(claw4_adj) and is_irreflexive(claw4_adj)))
}

/// The claw as a graph on four vertices.
///
/// A four-element vertex type is what `contains_induced` needs. The combinatorial condition in
/// `src/simple_graph_claw.ac` cuts its four vertices out of an ambient set, but an embedding
/// carries the *whole* vertex type of the forbidden graph, so the star on the naturals used in
/// `src/simple_graph_four_vertex.ac` would demand infinitely many leaves.
theorem claw4_graph_exists {
    exists(g: SimpleGraph[Fin[Nat.4]]) { SimpleGraph.new(claw4_adj) = Option.some(g) }
} by {
    claw4_adj_constraint
}

/// The claw on four vertices.
let claw4_graph: SimpleGraph[Fin[Nat.4]] satisfy {
    SimpleGraph.new(claw4_adj) = Option.some(claw4_graph)
}

/// Adjacency in the claw graph is having exactly one endpoint at the center.
theorem claw4_graph_adj_iff(x: Fin[Nat.4], y: Fin[Nat.4]) {
    claw4_graph.adj(x, y) = ((x.value = Nat.0 and y.value != Nat.0)
        or (y.value = Nat.0 and x.value != Nat.0))
} by {
    claw4_adj_constraint
    SimpleGraph.new(claw4_adj) = Option.some(claw4_graph)
    claw4_graph.adj = claw4_adj
    claw4_graph.adj(x, y) = claw4_adj(x, y)
    (claw4_adj(x, y) = ((x.value = Nat.0 and y.value != Nat.0)
        or (y.value = Nat.0 and x.value != Nat.0)))
}
