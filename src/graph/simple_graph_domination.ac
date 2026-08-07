from nat import Nat
from finite_set import FiniteSet, finite_set_subset_contains
from graph.simple_graph import SimpleGraph

numerals Nat

/// True if some member of `d` is adjacent to `v`.
define has_neighbor_in[V](g: SimpleGraph[V], d: FiniteSet[V], v: V) -> Bool {
    exists(w: V) {
        d.contains(w) and g.adj(w, v)
    }
}

/// A member of `d` adjacent to `v` witnesses `has_neighbor_in`.
theorem has_neighbor_in_intro[V](g: SimpleGraph[V], d: FiniteSet[V], v: V, w: V) {
    d.contains(w) and g.adj(w, v) implies has_neighbor_in(g, d, v)
} by {
    if d.contains(w) and g.adj(w, v) {
        has_neighbor_in(g, d, v) = exists(u: V) {
            d.contains(u) and g.adj(u, v)
        }
        exists(u: V) {
            d.contains(u) and g.adj(u, v)
        }
        has_neighbor_in(g, d, v)
    }
}

/// A neighbor in `d` can be extracted.
theorem has_neighbor_in_witness[V](g: SimpleGraph[V], d: FiniteSet[V], v: V) {
    has_neighbor_in(g, d, v) implies exists(w: V) { d.contains(w) and g.adj(w, v) }
} by {
    if has_neighbor_in(g, d, v) {
        has_neighbor_in(g, d, v) = exists(u: V) {
            d.contains(u) and g.adj(u, v)
        }
        exists(u: V) {
            d.contains(u) and g.adj(u, v)
        }
    }
}

/// True if `v` is either in `d` or adjacent to a member of `d`.
define is_dominated[V](g: SimpleGraph[V], d: FiniteSet[V], v: V) -> Bool {
    d.contains(v) or has_neighbor_in(g, d, v)
}

/// A member of the dominating set is dominated.
theorem is_dominated_of_contains[V](g: SimpleGraph[V], d: FiniteSet[V], v: V) {
    d.contains(v) implies is_dominated(g, d, v)
}

/// A vertex adjacent to a member of the dominating set is dominated.
theorem is_dominated_of_adj[V](g: SimpleGraph[V], d: FiniteSet[V], v: V, w: V) {
    d.contains(w) and g.adj(w, v) implies is_dominated(g, d, v)
} by {
    if d.contains(w) and g.adj(w, v) {
        has_neighbor_in_intro(g, d, v, w)
        has_neighbor_in(g, d, v)
        is_dominated(g, d, v)
    }
}

/// True if `d` dominates every vertex of the ambient set `s`.
define is_dominating_set[V](g: SimpleGraph[V], s: FiniteSet[V], d: FiniteSet[V]) -> Bool {
    forall(v: V) {
        s.contains(v) implies is_dominated(g, d, v)
    }
}

/// A vertex of the ambient set is dominated by a dominating set.
theorem is_dominating_set_apply[V](
    g: SimpleGraph[V], s: FiniteSet[V], d: FiniteSet[V], v: V
) {
    is_dominating_set(g, s, d) and s.contains(v) implies is_dominated(g, d, v)
} by {
    if is_dominating_set(g, s, d) and s.contains(v) {
        is_dominating_set(g, s, d) = forall(u: V) {
            s.contains(u) implies is_dominated(g, d, u)
        }
        forall(u: V) {
            s.contains(u) implies is_dominated(g, d, u)
        }
        s.contains(v) implies is_dominated(g, d, v)
        is_dominated(g, d, v)
    }
}

/// Pointwise domination of the ambient set is domination.
theorem is_dominating_set_intro[V](g: SimpleGraph[V], s: FiniteSet[V], d: FiniteSet[V]) {
    (forall(v: V) { s.contains(v) implies is_dominated(g, d, v) })
        implies is_dominating_set(g, s, d)
} by {
    if forall(v: V) { s.contains(v) implies is_dominated(g, d, v) } {
        is_dominating_set(g, s, d) = forall(u: V) {
            s.contains(u) implies is_dominated(g, d, u)
        }
        is_dominating_set(g, s, d)
    }
}

/// The whole ambient set dominates itself.
theorem is_dominating_set_self[V](g: SimpleGraph[V], s: FiniteSet[V]) {
    is_dominating_set(g, s, s)
} by {
    forall(v: V) {
        if s.contains(v) {
            is_dominated_of_contains(g, s, v)
            is_dominated(g, s, v)
        }
        s.contains(v) implies is_dominated(g, s, v)
    }
    is_dominating_set_intro(g, s, s)
    is_dominating_set(g, s, s)
}

/// Domination survives enlarging the dominating set.
theorem is_dominated_of_subset[V](
    g: SimpleGraph[V], c: FiniteSet[V], d: FiniteSet[V], v: V
) {
    c.subset_eq(d) and is_dominated(g, c, v) implies is_dominated(g, d, v)
} by {
    if c.subset_eq(d) and is_dominated(g, c, v) {
        if c.contains(v) {
            finite_set_subset_contains(c, d, v)
            d.contains(v)
            is_dominated_of_contains(g, d, v)
            is_dominated(g, d, v)
        }
        if not c.contains(v) {
            has_neighbor_in(g, c, v)
            has_neighbor_in_witness(g, c, v)
            let (w: V) satisfy {
                c.contains(w) and g.adj(w, v)
            }
            finite_set_subset_contains(c, d, w)
            d.contains(w)
            is_dominated_of_adj(g, d, v, w)
            is_dominated(g, d, v)
        }
        is_dominated(g, d, v)
    }
}

/// A superset of a dominating set is dominating.
theorem is_dominating_set_of_subset[V](
    g: SimpleGraph[V], s: FiniteSet[V], c: FiniteSet[V], d: FiniteSet[V]
) {
    c.subset_eq(d) and is_dominating_set(g, s, c) implies is_dominating_set(g, s, d)
} by {
    if c.subset_eq(d) and is_dominating_set(g, s, c) {
        forall(v: V) {
            if s.contains(v) {
                is_dominating_set_apply(g, s, c, v)
                is_dominated(g, c, v)
                is_dominated_of_subset(g, c, d, v)
                is_dominated(g, d, v)
            }
            s.contains(v) implies is_dominated(g, d, v)
        }
        is_dominating_set_intro(g, s, d)
        is_dominating_set(g, s, d)
    }
}

/// Shrinking the ambient set preserves domination.
theorem is_dominating_set_of_smaller_ambient[V](
    g: SimpleGraph[V], s: FiniteSet[V], t: FiniteSet[V], d: FiniteSet[V]
) {
    t.subset_eq(s) and is_dominating_set(g, s, d) implies is_dominating_set(g, t, d)
} by {
    if t.subset_eq(s) and is_dominating_set(g, s, d) {
        forall(v: V) {
            if t.contains(v) {
                finite_set_subset_contains(t, s, v)
                s.contains(v)
                is_dominating_set_apply(g, s, d, v)
                is_dominated(g, d, v)
            }
            t.contains(v) implies is_dominated(g, d, v)
        }
        is_dominating_set_intro(g, t, d)
        is_dominating_set(g, t, d)
    }
}
