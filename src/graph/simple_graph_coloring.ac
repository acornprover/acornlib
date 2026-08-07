/// Basic vertex colorings of simple graphs.

from data.basic.functions import compose, is_injective_fn, injective_fn_ne
from graph.simple_graph import SimpleGraph, SimpleGraphHom, complete_graph, complete_graph_adj_iff_ne,
    empty_graph, empty_graph_adj_false, is_graph_hom, simple_graph_hom_maps_adj

/// A proper vertex coloring: adjacent vertices receive distinct colors.
define simple_graph_coloring[V, C](g: SimpleGraph[V], color: V -> C) -> Bool {
    forall(x: V, y: V) {
        g.adj(x, y) implies color(x) != color(y)
    }
}

/// A proper coloring separates the colors of every adjacent pair.
theorem simple_graph_coloring_adj_ne[V, C](g: SimpleGraph[V], color: V -> C, x: V, y: V) {
    simple_graph_coloring(g, color) and g.adj(x, y) implies color(x) != color(y)
} by {
    if simple_graph_coloring(g, color) and g.adj(x, y) {
        simple_graph_coloring(g, color) = forall(a: V, b: V) {
            g.adj(a, b) implies color(a) != color(b)
        }
        color(x) != color(y)
    }
}

/// Every vertex map is a proper coloring of the empty graph.
theorem empty_graph_coloring[V, C](color: V -> C) {
    simple_graph_coloring(empty_graph[V], color)
} by {
    forall(x: V, y: V) {
        if empty_graph[V].adj(x, y) {
            empty_graph_adj_false[V](x, y)
            false
        }
    }
}

/// A coloring of the target graph pulls back along any graph homomorphism.
theorem simple_graph_hom_pullback_coloring[V, W, C](f: SimpleGraphHom[V, W], color: W -> C) {
    simple_graph_coloring(f.dst, color) implies simple_graph_coloring(f.src, compose(color, f.map))
} by {
    if simple_graph_coloring(f.dst, color) {
        forall(x: V, y: V) {
            if f.src.adj(x, y) {
                simple_graph_hom_maps_adj(f, x, y)
                f.dst.adj(f.map(x), f.map(y))
                simple_graph_coloring_adj_ne(f.dst, color, f.map(x), f.map(y))
                color(f.map(x)) != color(f.map(y))
                compose(color, f.map, x) = color(f.map(x))
                compose(color, f.map, y) = color(f.map(y))
                compose(color, f.map)(x) != compose(color, f.map)(y)
            }
        }
    }
}

/// A coloring of the complete graph is an injective function.
theorem complete_graph_coloring_imp_injective[V, C](color: V -> C) {
    simple_graph_coloring(complete_graph[V], color) implies is_injective_fn(color)
} by {
    if simple_graph_coloring(complete_graph[V], color) {
        forall(x: V, y: V) {
            if color(x) = color(y) {
                if x != y {
                    complete_graph_adj_iff_ne[V](x, y)
                    complete_graph[V].adj(x, y)
                    simple_graph_coloring_adj_ne(complete_graph[V], color, x, y)
                    color(x) != color(y)
                    false
                }
                x = y
            }
        }
    }
}

/// An injective function is a proper coloring of the complete graph.
theorem injective_imp_complete_graph_coloring[V, C](color: V -> C) {
    is_injective_fn(color) implies simple_graph_coloring(complete_graph[V], color)
} by {
    if is_injective_fn(color) {
        forall(x: V, y: V) {
            if complete_graph[V].adj(x, y) {
                complete_graph_adj_iff_ne[V](x, y)
                x != y
                injective_fn_ne(color, x, y)
                color(x) != color(y)
            }
        }
    }
}

/// Proper colorings are exactly graph homomorphisms to the complete graph on the color type.
theorem simple_graph_coloring_eq_graph_hom_to_complete[V, C](g: SimpleGraph[V], color: V -> C) {
    simple_graph_coloring(g, color) = is_graph_hom(g, complete_graph[C], color)
} by {
    if simple_graph_coloring(g, color) {
        forall(x: V, y: V) {
            if g.adj(x, y) {
                simple_graph_coloring_adj_ne(g, color, x, y)
                color(x) != color(y)
                complete_graph_adj_iff_ne[C](color(x), color(y))
                complete_graph[C].adj(color(x), color(y))
            }
        }
        is_graph_hom(g, complete_graph[C], color)
    }
    if is_graph_hom(g, complete_graph[C], color) {
        is_graph_hom(g, complete_graph[C], color) = forall(x: V, y: V) {
            g.adj(x, y) implies complete_graph[C].adj(color(x), color(y))
        }
        forall(x: V, y: V) {
            if g.adj(x, y) {
                complete_graph[C].adj(color(x), color(y))
                complete_graph_adj_iff_ne[C](color(x), color(y))
                color(x) != color(y)
            }
        }
        simple_graph_coloring(g, color)
    }
}
