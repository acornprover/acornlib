from nat import Nat, small_mod, div_imp_mod, divides_self, lt_and_lte
from data.basic.relation_basic import is_symmetric, is_irreflexive
from graph.simple_graph import SimpleGraph

numerals Nat

/// Adjacency on the cycle of length `n`: two residues are adjacent when consecutive modulo `n`.
///
/// The distinctness clause is needed as well as the successor clause: at `n = 1` every vertex
/// is its own successor modulo `n`, and a simple graph has no loops.
define cycle_adj(n: Nat, x: Nat, y: Nat) -> Bool {
    x != y and (x.suc.mod(n) = y or y.suc.mod(n) = x)
}

/// Consecutiveness modulo `n` is symmetric.
theorem cycle_adj_is_symmetric(n: Nat) {
    is_symmetric(cycle_adj(n))
} by {
    forall(x: Nat, y: Nat) {
        if cycle_adj(n, x, y) {
            cycle_adj(n, x, y) = (x != y and (x.suc.mod(n) = y or y.suc.mod(n) = x))
            x != y
            (x.suc.mod(n) = y or y.suc.mod(n) = x)
            y != x
            (y.suc.mod(n) = x or x.suc.mod(n) = y)
            cycle_adj(n, y, x) = (y != x and (y.suc.mod(n) = x or x.suc.mod(n) = y))
            cycle_adj(n, y, x)
        }
        (cycle_adj(n, x, y) implies cycle_adj(n, y, x))
    }
}

/// No vertex of the cycle is adjacent to itself.
theorem cycle_adj_is_irreflexive(n: Nat) {
    is_irreflexive(cycle_adj(n))
} by {
    forall(x: Nat) {
        if cycle_adj(n, x, x) {
            cycle_adj(n, x, x) = (x != x and (x.suc.mod(n) = x or x.suc.mod(n) = x))
            x != x
            false
        }
        not cycle_adj(n, x, x)
    }
}

/// `cycle_adj` satisfies the simple-graph constraint.
theorem cycle_adj_constraint(n: Nat) {
    is_symmetric(cycle_adj(n)) and is_irreflexive(cycle_adj(n))
} by {
    cycle_adj_is_symmetric(n)
    cycle_adj_is_irreflexive(n)
}

/// The cycle graph of length `n`, on the vertex set `{0, ..., n - 1}`.
///
/// As with the path, the graph is defined on all of the naturals and the finite cycle `C_n` is
/// cut out by the vertex set carried by every graph predicate. Outside `{0, ..., n - 1}` the
/// relation is not the intended one, which costs nothing: the vertex set is always supplied.
let cycle_graph(n: Nat) -> result: SimpleGraph[Nat] satisfy {
    SimpleGraph.new(cycle_adj(n)) = Option.some(result)
} by {
    cycle_adj_constraint(n)
}

/// Adjacency in the cycle graph is distinct consecutiveness modulo `n`.
theorem cycle_graph_adj_iff(n: Nat, x: Nat, y: Nat) {
    cycle_graph(n).adj(x, y) = (x != y and (x.suc.mod(n) = y or y.suc.mod(n) = x))
} by {
    cycle_graph(n).adj = cycle_adj(n)
    cycle_graph(n).adj(x, y) = cycle_adj(n, x, y)
    cycle_adj(n, x, y) = (x != y and (x.suc.mod(n) = y or y.suc.mod(n) = x))
}

/// Consecutive vertices inside the range are adjacent.
theorem cycle_graph_adj_suc(n: Nat, x: Nat) {
    x.suc < n implies cycle_graph(n).adj(x, x.suc)
} by {
    if x.suc < n {
        small_mod(x.suc, n)
        x.suc.mod(n) = x.suc
        x < x.suc
        x != x.suc
        cycle_graph_adj_iff(n, x, x.suc)
        (cycle_graph(n).adj(x, x.suc)
            = (x != x.suc and (x.suc.mod(n) = x.suc or x.suc.suc.mod(n) = x)))
        cycle_graph(n).adj(x, x.suc)
    }
}

/// The last vertex is adjacent to the first, which is what closes the cycle.
///
/// The hypothesis `2 <= x` makes the two ends distinct: on `C_1` and `C_2` the wrap-around
/// edge coincides with a vertex or with an edge already present.
theorem cycle_graph_adj_wrap(x: Nat) {
    Nat.2 <= x implies cycle_graph(x.suc).adj(x, Nat.0)
} by {
    if Nat.2 <= x {
        Nat.0 < Nat.2
        lt_and_lte(Nat.0, Nat.2, x)
        Nat.0 < x
        x != Nat.0
        divides_self(x.suc)
        div_imp_mod(x.suc, x.suc)
        x.suc.mod(x.suc) = Nat.0
        cycle_graph_adj_iff(x.suc, x, Nat.0)
        (cycle_graph(x.suc).adj(x, Nat.0)
            = (x != Nat.0 and (x.suc.mod(x.suc) = Nat.0 or Nat.0.suc.mod(x.suc) = x)))
        cycle_graph(x.suc).adj(x, Nat.0)
    }
}

/// Adjacent vertices of the cycle are distinct.
theorem cycle_graph_adj_ne(n: Nat, x: Nat, y: Nat) {
    cycle_graph(n).adj(x, y) implies x != y
} by {
    if cycle_graph(n).adj(x, y) {
        cycle_graph_adj_iff(n, x, y)
        (cycle_graph(n).adj(x, y) = (x != y and (x.suc.mod(n) = y or y.suc.mod(n) = x)))
        x != y
    }
}
