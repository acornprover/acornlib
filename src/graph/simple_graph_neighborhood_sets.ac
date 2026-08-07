from nat import Nat
from finite_set import FiniteSet, finite_set_subset_contains
from graph.simple_graph import SimpleGraph, simple_graph_adj_comm, simple_graph_adj_irreflexive
from graph.simple_graph_degree import neighborhood, neighborhood_contains_eq
from graph.simple_graph_independent import is_independent_in, is_independent_in_apply,
    is_clique_in, is_clique_in_apply
from graph.simple_graph_bipartite import is_bipartition, is_bipartition_apply
from graph.simple_graph_vertex_sets import is_vertex_cover, is_vertex_cover_apply

numerals Nat

/// Neighborhoods grow with the ambient vertex set.
theorem neighborhood_contains_of_ambient_subset[V](
    g: SimpleGraph[V], s: FiniteSet[V], t: FiniteSet[V], v: V, w: V
) {
    t.subset_eq(s) and neighborhood(g, t, v).contains(w) implies neighborhood(g, s, v).contains(w)
} by {
    if t.subset_eq(s) and neighborhood(g, t, v).contains(w) {
        neighborhood_contains_eq(g, t, v, w)
        t.contains(w) and g.adj(v, w)
        finite_set_subset_contains(t, s, w)
        s.contains(w)
        neighborhood_contains_eq(g, s, v, w)
        neighborhood(g, s, v).contains(w)
    }
}

/// An independent set meets no neighborhood of its own members.
theorem independent_neighborhood_empty[V](
    g: SimpleGraph[V], t: FiniteSet[V], v: V, w: V
) {
    is_independent_in(g, t) and t.contains(v) implies not neighborhood(g, t, v).contains(w)
} by {
    if is_independent_in(g, t) and t.contains(v) {
        if neighborhood(g, t, v).contains(w) {
            neighborhood_contains_eq(g, t, v, w)
            t.contains(w) and g.adj(v, w)
            is_independent_in_apply(g, t, v, w)
            not g.adj(v, w)
            false
        }
        not neighborhood(g, t, v).contains(w)
    }
}

/// In a clique, every other member is a neighbor.
theorem clique_neighborhood_contains[V](
    g: SimpleGraph[V], t: FiniteSet[V], v: V, w: V
) {
    is_clique_in(g, t) and t.contains(v) and t.contains(w) and v != w
        implies neighborhood(g, t, v).contains(w)
} by {
    if is_clique_in(g, t) and t.contains(v) and t.contains(w) and v != w {
        is_clique_in_apply(g, t, v, w)
        g.adj(v, w)
        neighborhood_contains_eq(g, t, v, w)
        neighborhood(g, t, v).contains(w)
    }
}

/// A neighbor of a vertex lies on the opposite side of any bipartition.
theorem bipartition_neighborhood_opposite[V](
    g: SimpleGraph[V], t: FiniteSet[V], part: V -> Bool, v: V, w: V
) {
    is_bipartition(g, t, part) and t.contains(v) and neighborhood(g, t, v).contains(w)
        implies part(v) != part(w)
} by {
    if is_bipartition(g, t, part) and t.contains(v) and neighborhood(g, t, v).contains(w) {
        neighborhood_contains_eq(g, t, v, w)
        t.contains(w) and g.adj(v, w)
        is_bipartition_apply(g, t, part, v, w)
        part(v) != part(w)
    }
}

/// A vertex outside a vertex cover has its whole neighborhood inside the cover.
theorem vertex_cover_neighborhood_contains[V](
    g: SimpleGraph[V], s: FiniteSet[V], c: FiniteSet[V], v: V, w: V
) {
    is_vertex_cover(g, s, c) and s.contains(v) and not c.contains(v)
        and neighborhood(g, s, v).contains(w) implies c.contains(w)
} by {
    if is_vertex_cover(g, s, c) and s.contains(v) and not c.contains(v) and neighborhood(g, s, v).contains(w) {
        neighborhood_contains_eq(g, s, v, w)
        s.contains(w) and g.adj(v, w)
        is_vertex_cover_apply(g, s, c, v, w)
        c.contains(v) or c.contains(w)
        c.contains(w)
    }
}

/// Neighborhood membership is symmetric within the ambient set.
theorem neighborhood_symmetric[V](g: SimpleGraph[V], s: FiniteSet[V], v: V, w: V) {
    s.contains(v) and neighborhood(g, s, v).contains(w) implies neighborhood(g, s, w).contains(v)
} by {
    if s.contains(v) and neighborhood(g, s, v).contains(w) {
        neighborhood_contains_eq(g, s, v, w)
        s.contains(w) and g.adj(v, w)
        simple_graph_adj_comm(g, v, w)
        g.adj(w, v)
        neighborhood_contains_eq(g, s, w, v)
        neighborhood(g, s, w).contains(v)
    }
}

/// A vertex is never in its own neighborhood.
theorem neighborhood_excludes_center[V](g: SimpleGraph[V], s: FiniteSet[V], v: V) {
    not neighborhood(g, s, v).contains(v)
} by {
    if neighborhood(g, s, v).contains(v) {
        neighborhood_contains_eq(g, s, v, v)
        s.contains(v) and g.adj(v, v)
        simple_graph_adj_irreflexive(g, v)
        not g.adj(v, v)
        false
    }
    not neighborhood(g, s, v).contains(v)
}

/// Two vertices on the same side of a bipartition are never neighbors.
theorem bipartition_same_side_not_neighbor[V](
    g: SimpleGraph[V], t: FiniteSet[V], part: V -> Bool, v: V, w: V
) {
    is_bipartition(g, t, part) and t.contains(v) and part(v) = part(w)
        implies not neighborhood(g, t, v).contains(w)
} by {
    if is_bipartition(g, t, part) and t.contains(v) and part(v) = part(w) {
        if neighborhood(g, t, v).contains(w) {
            bipartition_neighborhood_opposite(g, t, part, v, w)
            part(v) != part(w)
            false
        }
        not neighborhood(g, t, v).contains(w)
    }
}
