from finite_set import FiniteSet
from graph.simple_graph import SimpleGraph, graph_complement, graph_complement_adj_eq,
    simple_graph_adj_irreflexive
from graph.simple_graph_independent import is_independent_in, is_independent_in_apply,
    is_independent_in_intro, is_clique_in, is_clique_in_apply, is_clique_in_intro

/// A clique of a graph is an independent set of its complement.
///
/// The complement joins exactly the distinct non-adjacent pairs, so being adjacent in the graph
/// and being non-adjacent in the complement say the same thing about a distinct pair. The equal
/// pair is handled by irreflexivity on both sides.
theorem clique_is_independent_in_complement[V](g: SimpleGraph[V], t: FiniteSet[V]) {
    is_clique_in(g, t) implies is_independent_in(graph_complement(g), t)
} by {
    if is_clique_in(g, t) {
        forall(x: V, y: V) {
            if t.contains(x) and t.contains(y) {
                graph_complement_adj_eq(g, x, y)
                (graph_complement(g).adj(x, y) = (x != y and not g.adj(x, y)))
                if x = y {
                    not (x != y)
                    not graph_complement(g).adj(x, y)
                }
                if x != y {
                    is_clique_in_apply(g, t, x, y)
                    g.adj(x, y)
                    not (not g.adj(x, y))
                    not graph_complement(g).adj(x, y)
                }
                not graph_complement(g).adj(x, y)
            }
            (t.contains(x) and t.contains(y) implies not graph_complement(g).adj(x, y))
        }
        is_independent_in_intro(graph_complement(g), t)
        is_independent_in(graph_complement(g), t)
    }
}

/// An independent set of the complement is a clique of the graph.
theorem independent_in_complement_is_clique[V](g: SimpleGraph[V], t: FiniteSet[V]) {
    is_independent_in(graph_complement(g), t) implies is_clique_in(g, t)
} by {
    if is_independent_in(graph_complement(g), t) {
        forall(x: V, y: V) {
            if t.contains(x) and t.contains(y) and x != y {
                is_independent_in_apply(graph_complement(g), t, x, y)
                not graph_complement(g).adj(x, y)
                graph_complement_adj_eq(g, x, y)
                (graph_complement(g).adj(x, y) = (x != y and not g.adj(x, y)))
                not (x != y and not g.adj(x, y))
                g.adj(x, y)
            }
            (t.contains(x) and t.contains(y) and x != y implies g.adj(x, y))
        }
        is_clique_in_intro(g, t)
        is_clique_in(g, t)
    }
}

/// Cliques of a graph are exactly independent sets of its complement.
theorem clique_iff_independent_in_complement[V](g: SimpleGraph[V], t: FiniteSet[V]) {
    is_clique_in(g, t) = is_independent_in(graph_complement(g), t)
} by {
    if is_clique_in(g, t) {
        clique_is_independent_in_complement(g, t)
        is_independent_in(graph_complement(g), t)
    }
    if is_independent_in(graph_complement(g), t) {
        independent_in_complement_is_clique(g, t)
        is_clique_in(g, t)
    }
    (is_clique_in(g, t) implies is_independent_in(graph_complement(g), t))
    (is_independent_in(graph_complement(g), t) implies is_clique_in(g, t))
    is_clique_in(g, t) = is_independent_in(graph_complement(g), t)
}

/// An independent set of a graph is a clique of its complement.
theorem independent_is_clique_in_complement[V](g: SimpleGraph[V], t: FiniteSet[V]) {
    is_independent_in(g, t) implies is_clique_in(graph_complement(g), t)
} by {
    if is_independent_in(g, t) {
        forall(x: V, y: V) {
            if t.contains(x) and t.contains(y) and x != y {
                is_independent_in_apply(g, t, x, y)
                not g.adj(x, y)
                graph_complement_adj_eq(g, x, y)
                (graph_complement(g).adj(x, y) = (x != y and not g.adj(x, y)))
                graph_complement(g).adj(x, y)
            }
            (t.contains(x) and t.contains(y) and x != y
                implies graph_complement(g).adj(x, y))
        }
        is_clique_in_intro(graph_complement(g), t)
        is_clique_in(graph_complement(g), t)
    }
}

/// A clique of the complement is an independent set of the graph.
theorem clique_in_complement_is_independent[V](g: SimpleGraph[V], t: FiniteSet[V]) {
    is_clique_in(graph_complement(g), t) implies is_independent_in(g, t)
} by {
    if is_clique_in(graph_complement(g), t) {
        forall(x: V, y: V) {
            if t.contains(x) and t.contains(y) {
                if x = y {
                    simple_graph_adj_irreflexive(g, x)
                    not g.adj(x, x)
                    not g.adj(x, y)
                }
                if x != y {
                    is_clique_in_apply(graph_complement(g), t, x, y)
                    graph_complement(g).adj(x, y)
                    graph_complement_adj_eq(g, x, y)
                    (graph_complement(g).adj(x, y) = (x != y and not g.adj(x, y)))
                    not g.adj(x, y)
                }
                not g.adj(x, y)
            }
            (t.contains(x) and t.contains(y) implies not g.adj(x, y))
        }
        is_independent_in_intro(g, t)
        is_independent_in(g, t)
    }
}

/// Independent sets of a graph are exactly cliques of its complement.
///
/// The other half of the duality. Together with the previous form this is what lets a statement
/// about independence be read as one about cliques and back, which is how the independence
/// number and the clique number of complementary graphs are related.
theorem independent_iff_clique_in_complement[V](g: SimpleGraph[V], t: FiniteSet[V]) {
    is_independent_in(g, t) = is_clique_in(graph_complement(g), t)
} by {
    if is_independent_in(g, t) {
        independent_is_clique_in_complement(g, t)
        is_clique_in(graph_complement(g), t)
    }
    if is_clique_in(graph_complement(g), t) {
        clique_in_complement_is_independent(g, t)
        is_independent_in(g, t)
    }
    (is_independent_in(g, t) implies is_clique_in(graph_complement(g), t))
    (is_clique_in(graph_complement(g), t) implies is_independent_in(g, t))
    is_independent_in(g, t) = is_clique_in(graph_complement(g), t)
}
