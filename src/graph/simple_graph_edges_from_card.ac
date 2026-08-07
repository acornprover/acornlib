from nat import Nat
from pair import Pair
from data.basic.functions import is_injective_fn
from finite_set import FiniteSet, fs_image, finite_set_image_contains_eq,
    finite_set_image_cardinality_is_of_injective, finite_set_ext_contains,
    finite_set_cardinality_is_well_defined
from data.finite.finite_set_card import fs_card, fs_card_eq_of_cardinality_is, fs_card_cardinality_is
from graph.simple_graph import SimpleGraph
from graph.simple_graph_degree import degree, neighborhood, neighborhood_contains_eq
from graph.simple_graph_degree_sum import directed_edges_from, directed_edges_from_contains_eq,
    directed_edges_from_contains_pair, directed_edges_from_second_is_neighbor
from graph.simple_graph_edges import directed_edges, directed_edges_contains_eq

numerals Nat

/// Pairing a fixed first component with a varying second.
define pair_with[V](v: V) -> (V -> Pair[V, V]) {
    function(w: V) {
        Pair.new(v, w)
    }
}

/// The pairing map produces the expected pair.
theorem pair_with_eq[V](v: V, w: V) {
    pair_with(v)(w) = Pair.new(v, w)
}

/// Pairing with a fixed first component is injective.
///
/// Two pairs with the same first component agree exactly when their second
/// components do.
theorem pair_with_injective[V](v: V) {
    is_injective_fn(pair_with(v))
} by {
    forall(x: V, y: V) {
        if pair_with(v)(x) = pair_with(v)(y) {
            pair_with_eq(v, x)
            pair_with_eq(v, y)
            Pair.new(v, x) = Pair.new(v, y)
            Pair.new(v, x).second = x
            Pair.new(v, y).second = y
            x = y
        }
        pair_with(v)(x) = pair_with(v)(y) implies x = y
    }
    is_injective_fn(pair_with(v))
}

/// The edges leaving `v` are the image of its neighborhood under pairing with `v`.
///
/// Every neighbor gives an edge, and every edge leaving `v` comes from its endpoint,
/// so the two finite sets have the same members.
theorem directed_edges_from_is_image[V](
    g: SimpleGraph[V], s: FiniteSet[V], v: V
) {
    s.contains(v) implies directed_edges_from(g, s, v) = fs_image(neighborhood(g, s, v), pair_with(v))
} by {
    if s.contains(v) {
        forall(p: Pair[V, V]) {
            if directed_edges_from(g, s, v).contains(p) {
                directed_edges_from_second_is_neighbor(g, s, v, p)
                neighborhood(g, s, v).contains(p.second)
                directed_edges_from_contains_eq(g, s, v, p)
                p.first = v
                pair_with_eq(v, p.second)
                pair_with(v)(p.second) = Pair.new(v, p.second)
                Pair.new(p.first, p.second) = p
                p = pair_with(v)(p.second)
                finite_set_image_contains_eq(neighborhood(g, s, v), pair_with(v), p)
                fs_image(neighborhood(g, s, v), pair_with(v)).contains(p)
            }
            if fs_image(neighborhood(g, s, v), pair_with(v)).contains(p) {
                finite_set_image_contains_eq(neighborhood(g, s, v), pair_with(v), p)
                let (w: V) satisfy {
                    neighborhood(g, s, v).contains(w) and p = pair_with(v)(w)
                }
                directed_edges_from_contains_pair(g, s, v, w)
                directed_edges_from(g, s, v).contains(Pair.new(v, w))
                pair_with_eq(v, w)
                p = Pair.new(v, w)
                directed_edges_from(g, s, v).contains(p)
            }
            directed_edges_from(g, s, v).contains(p) = fs_image(neighborhood(g, s, v), pair_with(v)).contains(p)
            directed_edges_from(g, s, v).contains(p) = directed_edges_from(g, s, v).underlying_set.contains(p)
            fs_image(neighborhood(g, s, v), pair_with(v)).contains(p) = fs_image(neighborhood(g, s, v), pair_with(v)).underlying_set.contains(p)
            directed_edges_from(g, s, v).underlying_set.contains(p) = fs_image(neighborhood(g, s, v), pair_with(v)).underlying_set.contains(p)
        }
        finite_set_ext_contains(directed_edges_from(g, s, v), fs_image(neighborhood(g, s, v), pair_with(v)))
        directed_edges_from(g, s, v) = fs_image(neighborhood(g, s, v), pair_with(v))
    }
}

/// The edges leaving `v` are as numerous as its neighborhood.
theorem directed_edges_from_cardinality_is[V](g: SimpleGraph[V], s: FiniteSet[V], v: V) {
    s.contains(v) implies directed_edges_from(g, s, v).cardinality_is(fs_card(neighborhood(g, s, v)))
} by {
    if s.contains(v) {
        fs_card_cardinality_is(neighborhood(g, s, v))
        neighborhood(g, s, v).cardinality_is(fs_card(neighborhood(g, s, v)))
        pair_with_injective(v)
        is_injective_fn(pair_with(v))
        finite_set_image_cardinality_is_of_injective(
            neighborhood(g, s, v), pair_with(v), fs_card(neighborhood(g, s, v))
        )
        fs_image(neighborhood(g, s, v), pair_with(v)).cardinality_is(fs_card(neighborhood(g, s, v)))
        directed_edges_from_is_image(g, s, v)
        directed_edges_from(g, s, v) = fs_image(neighborhood(g, s, v), pair_with(v))
        directed_edges_from(g, s, v).cardinality_is(fs_card(neighborhood(g, s, v)))
    }
}

/// A vertex has exactly as many outgoing directed edges as it has neighbors.
///
/// This is the per-vertex half of the handshake lemma: summing it over the vertex
/// set gives the degree sum on one side and the directed edge count on the other.
theorem directed_edges_from_card[V](g: SimpleGraph[V], s: FiniteSet[V], v: V) {
    s.contains(v) implies fs_card(directed_edges_from(g, s, v)) = degree(g, s, v)
} by {
    if s.contains(v) {
        directed_edges_from_cardinality_is(g, s, v)
        directed_edges_from(g, s, v).cardinality_is(fs_card(neighborhood(g, s, v)))
        fs_card_eq_of_cardinality_is(directed_edges_from(g, s, v), fs_card(neighborhood(g, s, v)))
        fs_card(directed_edges_from(g, s, v)) = fs_card(neighborhood(g, s, v))
        degree(g, s, v) = fs_card(neighborhood(g, s, v))
        fs_card(directed_edges_from(g, s, v)) = degree(g, s, v)
    }
}
