from data.basic.functions import compose, is_injective_fn, injective_fn_ne
from data.basic.logic import iff_bool, iff_bool_imp_eq
from graph.simple_graph import SimpleGraph
from graph.simple_graph_coloring import simple_graph_coloring, simple_graph_coloring_adj_ne

/// Injectively relabeling colors preserves a proper coloring.
theorem simple_graph_coloring_injective_color_relabel[V, C, D](
    g: SimpleGraph[V],
    color: V -> C,
    relabel: C -> D
) {
    simple_graph_coloring(g, color) and is_injective_fn(relabel)
    implies simple_graph_coloring(g, compose(relabel, color))
} by {
    if simple_graph_coloring(g, color) and is_injective_fn(relabel) {
        forall(x: V, y: V) {
            if g.adj(x, y) {
                simple_graph_coloring_adj_ne(g, color, x, y)
                color(x) != color(y)
                injective_fn_ne(relabel, color(x), color(y))
                relabel(color(x)) != relabel(color(y))
                compose(relabel, color, x) = relabel(color(x))
                compose(relabel, color, y) = relabel(color(y))
                compose(relabel, color)(x) != compose(relabel, color)(y)
            }
        }
        simple_graph_coloring(g, compose(relabel, color))
    }
}

/// If a relabeled map is a proper coloring, then the original map is a proper coloring.
theorem simple_graph_coloring_of_color_relabel[V, C, D](
    g: SimpleGraph[V],
    color: V -> C,
    relabel: C -> D
) {
    simple_graph_coloring(g, compose(relabel, color)) implies
    simple_graph_coloring(g, color)
} by {
    if simple_graph_coloring(g, compose(relabel, color)) {
        forall(x: V, y: V) {
            if g.adj(x, y) {
                simple_graph_coloring_adj_ne(g, compose(relabel, color), x, y)
                compose(relabel, color)(x) != compose(relabel, color)(y)
                compose(relabel, color, x) = relabel(color(x))
                compose(relabel, color, y) = relabel(color(y))
                relabel(color(x)) != relabel(color(y))
                if color(x) = color(y) {
                    relabel(color(x)) = relabel(color(y))
                    false
                }
                color(x) != color(y)
            }
        }
        simple_graph_coloring(g, color)
    }
}

/// Injective color relabeling preserves and reflects proper colorings.
theorem simple_graph_coloring_color_relabel_iff_of_injective[V, C, D](
    g: SimpleGraph[V],
    color: V -> C,
    relabel: C -> D
) {
    is_injective_fn(relabel) implies
    simple_graph_coloring(g, compose(relabel, color)) = simple_graph_coloring(g, color)
} by {
    if is_injective_fn(relabel) {
        if simple_graph_coloring(g, compose(relabel, color)) {
            simple_graph_coloring_of_color_relabel(g, color, relabel)
            simple_graph_coloring(g, color)
        }
        if simple_graph_coloring(g, color) {
            simple_graph_coloring_injective_color_relabel(g, color, relabel)
            simple_graph_coloring(g, compose(relabel, color))
        }
        iff_bool(
            simple_graph_coloring(g, compose(relabel, color)),
            simple_graph_coloring(g, color)
        )
        iff_bool_imp_eq(
            simple_graph_coloring(g, compose(relabel, color)),
            simple_graph_coloring(g, color)
        )
        simple_graph_coloring(g, compose(relabel, color)) = simple_graph_coloring(g, color)
    }
}
