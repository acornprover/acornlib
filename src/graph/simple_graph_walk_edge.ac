/// The directed edges traversed by a walk, and the induced notion of consecutiveness along a
/// cycle.
///
/// A walk is stored as its start vertex and its list of edge targets, matching
/// `simple_graph_walk`: the traversed directed edges are `(start, head(steps))` and then the
/// pairs of consecutive targets. This file packages the elementary facts about that relation
/// with a minimal import surface, so that proof search for the basic unfolding steps stays
/// cheap; the chordal-graph development builds on it.

from nat import Nat
from list import List
from data.list.list_cons_membership import cons_contains_eq, cons_contains_head,
    cons_contains_of_tail_contains, cons_contains_cases, nil_not_contains

numerals Nat


/// True when the walk `(start, steps)` traverses the directed edge `(x, y)`.
///
/// The walk is stored as its start vertex and the list of edge targets, matching
/// `simple_graph_walk`: the traversed directed edges are `(start, head(steps))` and then the
/// pairs of consecutive targets. In a simple cycle every vertex occurs once, so this is the
/// right notion of consecutiveness along the cycle: two distinct vertices of the cycle are
/// consecutive exactly when the walk traverses one of the two directed edges between them.
define is_walk_edge[V](start: V, steps: List[V], x: V, y: V) -> Bool {
    match steps {
        List.nil[V] {
            false
        }
        List.cons(next, rest) {
            (start = x and next = y) or is_walk_edge(next, rest, x, y)
        }
    }
}

/// The empty walk traverses no directed edge.
theorem is_walk_edge_nil[V](start: V, x: V, y: V) {
    is_walk_edge(start, List.nil[V], x, y) = false
} by {
    if is_walk_edge(start, List.nil[V], x, y) {
        false
    }
    if false {
        is_walk_edge(start, List.nil[V], x, y)
    }
}

/// A walk of one edge traverses exactly its own directed edge, or a later one: the forward
/// direction of the unfolding.
theorem is_walk_edge_dir1[V](start: V, next: V, rest: List[V], x: V, y: V) {
    is_walk_edge(start, List.cons(next, rest), x, y) implies
        ((start = x and next = y) or is_walk_edge(next, rest, x, y))
} by {
    if is_walk_edge(start, List.cons(next, rest), x, y) {
        (start = x and next = y) or is_walk_edge(next, rest, x, y)
    }
}

/// A walk of one edge traverses exactly its own directed edge, or a later one: the reverse
/// direction of the unfolding.
theorem is_walk_edge_dir2[V](start: V, next: V, rest: List[V], x: V, y: V) {
    ((start = x and next = y) or is_walk_edge(next, rest, x, y)) implies
        is_walk_edge(start, List.cons(next, rest), x, y)
} by {
    if (start = x and next = y) or is_walk_edge(next, rest, x, y) {
        if start = x and next = y {
            is_walk_edge(start, List.cons(next, rest), x, y)
        }
        if not (start = x and next = y) {
            if (start = x and next = y) or is_walk_edge(next, rest, x, y) {
                if start = x and next = y {
                    false
                }
                if not (start = x and next = y) {
                    is_walk_edge(next, rest, x, y)
                }
                is_walk_edge(next, rest, x, y)
            }
            is_walk_edge(next, rest, x, y)
            is_walk_edge(start, List.cons(next, rest), x, y)
        }
        is_walk_edge(start, List.cons(next, rest), x, y)
    }
}

/// A walk of one edge traverses exactly its own directed edge, or a later one.
theorem is_walk_edge_cons[V](start: V, next: V, rest: List[V], x: V, y: V) {
    is_walk_edge(start, List.cons(next, rest), x, y) =
        ((start = x and next = y) or is_walk_edge(next, rest, x, y))
} by {
    is_walk_edge_dir1(start, next, rest, x, y)
    is_walk_edge(start, List.cons(next, rest), x, y) implies ((start = x and next = y) or is_walk_edge(next, rest, x, y))
    is_walk_edge_dir2(start, next, rest, x, y)
    ((start = x and next = y) or is_walk_edge(next, rest, x, y)) implies is_walk_edge(start, List.cons(next, rest), x, y)
    if is_walk_edge(start, List.cons(next, rest), x, y) {
        (start = x and next = y) or is_walk_edge(next, rest, x, y)
        is_walk_edge(start, List.cons(next, rest), x, y) = ((start = x and next = y) or is_walk_edge(next, rest, x, y))
    }
    if not is_walk_edge(start, List.cons(next, rest), x, y) {
        if (start = x and next = y) or is_walk_edge(next, rest, x, y) {
            is_walk_edge(start, List.cons(next, rest), x, y)
            false
        }
        not ((start = x and next = y) or is_walk_edge(next, rest, x, y))
        is_walk_edge(start, List.cons(next, rest), x, y) = ((start = x and next = y) or is_walk_edge(next, rest, x, y))
    }
    is_walk_edge(start, List.cons(next, rest), x, y) = ((start = x and next = y) or is_walk_edge(next, rest, x, y))
}

/// A directed edge `(x, y)` of a walk whose target list contains no `y` cannot occur: every
/// traversed edge points at a target.
///
/// The condition is exactly that `y` occurs nowhere among the targets, so the walk can never
/// step onto `y`.
theorem is_walk_edge_false_of_second_absent[V](prev: V, xs: List[V], x: V, y: V) {
    (forall(w: V) { xs.contains(w) implies w != y }) implies not is_walk_edge(prev, xs, x, y)
} by {
    define p(ys: List[V]) -> Bool {
        forall(pv: V, a: V) {
            (forall(w: V) { ys.contains(w) implies w != y }) implies not is_walk_edge(pv, ys, a, y)
        }
    }

    forall(pv: V, a: V) {
        if forall(w: V) { List.nil[V].contains(w) implies w != y } {
            if is_walk_edge(pv, List.nil[V], a, y) {
                false
            }
            not is_walk_edge(pv, List.nil[V], a, y)
        }
        (forall(w: V) { List.nil[V].contains(w) implies w != y }) implies not is_walk_edge(pv, List.nil[V], a, y)
    }
    p(List.nil[V])

    forall(next: V, tail: List[V]) {
        if p(tail) {
            forall(pv: V, a: V) {
                if forall(w: V) { List.cons(next, tail).contains(w) implies w != y } {
                    p(tail)
                    p(tail) = forall(u: V, b: V) {
                        (forall(w: V) { tail.contains(w) implies w != y }) implies not is_walk_edge(u, tail, b, y)
                    }
                    (forall(w: V) { tail.contains(w) implies w != y }) implies not is_walk_edge(next, tail, a, y)
                    not is_walk_edge(next, tail, a, y)
                    if is_walk_edge(pv, List.cons(next, tail), a, y) {
                        is_walk_edge_cons(pv, next, tail, a, y)
                        is_walk_edge(pv, List.cons(next, tail), a, y) =
                            ((pv = a and next = y) or is_walk_edge(next, tail, a, y))
                        if pv = a and next = y {
                            next = y
                            next != y
                            false
                        }
                        if not (pv = a and next = y) {
                            false
                        }
                        false
                    }
                    not is_walk_edge(pv, List.cons(next, tail), a, y)
                }
                (forall(w: V) { List.cons(next, tail).contains(w) implies w != y }) implies not is_walk_edge(pv, List.cons(next, tail), a, y)
            }
            p(List.cons(next, tail))
        }
    }
    forall(next: V, tail: List[V]) { p(tail) implies p(List.cons(next, tail)) }
    function[T0](x0: List[T0] -> Bool) {
        not forall(x1: T0, x2: List[T0]) { not x0(x2) or x0(List.cons(x1, x2)) } or
        not x0(List.nil[T0]) or forall(x3: List[T0]) { x0(x3) } = true
    }[V](p)
    function[T0](x0: List[T0] -> Bool, x1: List[T0]) {
        not forall(x2: T0, x3: List[T0]) { not x0(x3) or x0(List.cons(x2, x3)) } or
        not x0(List.nil[T0]) or x0(x1)
    }[V](p, xs)
    p(xs)
    if forall(w: V) { xs.contains(w) implies w != y } {
        p(xs) = forall(u: V, b: V) {
            (forall(w: V) { xs.contains(w) implies w != y }) implies not is_walk_edge(u, xs, b, y)
        }
        (forall(w: V) { xs.contains(w) implies w != y }) implies not is_walk_edge(prev, xs, x, y)
        not is_walk_edge(prev, xs, x, y)
    }
}

/// A directed edge `(x, y)` of a walk whose target list contains no `x` can only be the
/// first edge, so a first vertex different from `x` rules it out entirely.
theorem is_walk_edge_false_of_first_absent[V](prev: V, xs: List[V], x: V, y: V) {
    (forall(w: V) { xs.contains(w) implies w != x }) and prev != x implies not is_walk_edge(prev, xs, x, y)
} by {
    define p(ys: List[V]) -> Bool {
        forall(pv: V, b: V) {
            (forall(w: V) { ys.contains(w) implies w != x }) and pv != x implies not is_walk_edge(pv, ys, x, b)
        }
    }

    forall(pv: V, b: V) {
        if (forall(w: V) { List.nil[V].contains(w) implies w != x }) and pv != x {
            if is_walk_edge(pv, List.nil[V], x, b) {
                false
            }
            not is_walk_edge(pv, List.nil[V], x, b)
        }
        (forall(w: V) { List.nil[V].contains(w) implies w != x }) and pv != x implies not is_walk_edge(pv, List.nil[V], x, b)
    }
    p(List.nil[V])

    forall(next: V, tail: List[V]) {
        if p(tail) {
            forall(pv: V, b: V) {
                if (forall(w: V) { List.cons(next, tail).contains(w) implies w != x }) and pv != x {
                    pv != x
                    cons_contains_head(next, tail)
                    List.cons(next, tail).contains(next)
                    List.cons(next, tail).contains(next) implies next != x
                    next != x
                    forall(w: V) {
                        if tail.contains(w) {
                            cons_contains_of_tail_contains(next, tail, w)
                            List.cons(next, tail).contains(w)
                            List.cons(next, tail).contains(w) implies w != x
                            w != x
                        }
                        tail.contains(w) implies w != x
                    }
                    p(tail)
                    p(tail) = forall(u: V, c: V) {
                        (forall(w: V) { tail.contains(w) implies w != x }) and u != x implies not is_walk_edge(u, tail, x, c)
                    }
                    (forall(w: V) { tail.contains(w) implies w != x }) and next != x implies not is_walk_edge(next, tail, x, b)
                    (forall(w: V) { tail.contains(w) implies w != x }) and next != x
                    not is_walk_edge(next, tail, x, b)
                    if is_walk_edge(pv, List.cons(next, tail), x, b) {
                        is_walk_edge_cons(pv, next, tail, x, b)
                        is_walk_edge(pv, List.cons(next, tail), x, b) =
                            ((pv = x and next = b) or is_walk_edge(next, tail, x, b))
                        if pv = x and next = b {
                            pv = x
                            pv != x
                            false
                        }
                        if not (pv = x and next = b) {
                            false
                        }
                        false
                    }
                    not is_walk_edge(pv, List.cons(next, tail), x, b)
                }
                (forall(w: V) { List.cons(next, tail).contains(w) implies w != x }) and pv != x implies not is_walk_edge(pv, List.cons(next, tail), x, b)
            }
            p(List.cons(next, tail))
        }
    }
    forall(next: V, tail: List[V]) { p(tail) implies p(List.cons(next, tail)) }
    function[T0](x0: List[T0] -> Bool) {
        not forall(x1: T0, x2: List[T0]) { not x0(x2) or x0(List.cons(x1, x2)) } or
        not x0(List.nil[T0]) or forall(x3: List[T0]) { x0(x3) } = true
    }[V](p)
    function[T0](x0: List[T0] -> Bool, x1: List[T0]) {
        not forall(x2: T0, x3: List[T0]) { not x0(x3) or x0(List.cons(x2, x3)) } or
        not x0(List.nil[T0]) or x0(x1)
    }[V](p, xs)
    p(xs)
    if (forall(w: V) { xs.contains(w) implies w != x }) and prev != x {
        p(xs) = forall(u: V, c: V) {
            (forall(w: V) { xs.contains(w) implies w != x }) and u != x implies not is_walk_edge(u, xs, x, c)
        }
        (forall(w: V) { xs.contains(w) implies w != x }) and prev != x implies not is_walk_edge(prev, xs, x, y)
        not is_walk_edge(prev, xs, x, y)
    }
}
