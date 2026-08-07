from nat import Nat, lt_and_lte
from data.fin.fin import Fin, new_round_trip, fin_new_exists_of_lt
from data.basic.functions import is_injective_fn, injective_fn_eq
from finite_set import FiniteSet, fs_insert
from data.finite.finite_set_insert_members import fs_insert_contains_item, fs_insert_contains_of_contains
from graph.simple_graph import SimpleGraph, is_graph_embedding, graph_embedding_is_injective
from graph.simple_graph_degree import neighborhood, neighborhood_contains_eq
from graph.simple_graph_claw import has_independent_neighbor_triple,
    has_independent_neighbor_triple_intro, is_claw_free, is_claw_free_apply
from graph.simple_graph_claw_graph import claw4_graph, claw4_graph_adj_iff
from graph.simple_graph_claw_embed import claw4_eq_of_value_eq, is_claw_free_of_free_of_claw
from graph.simple_graph_forbidden import contains_induced, contains_induced_witness, is_free_of,
    is_free_of_intro, graph_embedding_adj_forward, graph_embedding_adj_backward

numerals Nat

/// Each of the four labels names a claw vertex.
theorem claw4_witness(k: Nat) {
    k < Nat.4 implies exists(i: Fin[Nat.4]) { i.value = k }
} by {
    if k < Nat.4 {
        fin_new_exists_of_lt(Nat.4, k)
        exists(i: Fin[Nat.4]) { Fin[Nat.4].new(k) = Option.some(i) }
        let (j: Fin[Nat.4]) satisfy {
            Fin[Nat.4].new(k) = Option.some(j)
        }
        new_round_trip(Nat.4, k, j)
        j.value = k
        exists(i: Fin[Nat.4]) { i.value = k }
    }
}

/// The four labels are below four and pairwise distinct.
///
/// Comparisons between numerals are not free, so the ladder is stated once and everything
/// downstream reads the distinctness off it.
theorem claw4_labels {
    Nat.0 < Nat.4 and Nat.1 < Nat.4 and Nat.2 < Nat.4 and Nat.3 < Nat.4
        and Nat.0 != Nat.1 and Nat.0 != Nat.2 and Nat.0 != Nat.3
        and Nat.1 != Nat.2 and Nat.1 != Nat.3 and Nat.2 != Nat.3
} by {
    (Nat.0.suc = Nat.1)
    Nat.0 < Nat.1
    (Nat.1.suc = Nat.2)
    Nat.1 < Nat.2
    (Nat.2.suc = Nat.3)
    Nat.2 < Nat.3
    (Nat.3.suc = Nat.4)
    Nat.3 < Nat.4
    Nat.3 <= Nat.4
    lt_and_lte(Nat.2, Nat.3, Nat.4)
    Nat.2 < Nat.4
    Nat.2 <= Nat.4
    lt_and_lte(Nat.1, Nat.2, Nat.4)
    Nat.1 < Nat.4
    Nat.1 <= Nat.4
    lt_and_lte(Nat.0, Nat.1, Nat.4)
    Nat.0 < Nat.4
    Nat.2 <= Nat.3
    lt_and_lte(Nat.1, Nat.2, Nat.3)
    Nat.1 < Nat.3
    Nat.1 <= Nat.2
    lt_and_lte(Nat.0, Nat.1, Nat.2)
    Nat.0 < Nat.2
    Nat.1 <= Nat.3
    lt_and_lte(Nat.0, Nat.1, Nat.3)
    Nat.0 < Nat.3
    Nat.0 != Nat.1
    Nat.0 != Nat.2
    Nat.0 != Nat.3
    Nat.1 != Nat.2
    Nat.1 != Nat.3
    Nat.2 != Nat.3
    (Nat.0 != Nat.1 and Nat.0 != Nat.2 and Nat.0 != Nat.3
        and Nat.1 != Nat.2 and Nat.1 != Nat.3 and Nat.2 != Nat.3)
}

/// The four-element set holding a claw configuration.
///
/// A vertex set is needed only because the combinatorial condition carries one; any set holding
/// the four vertices does, and this is the smallest.
define claw_witness_set[V](a: V, b: V, c: V, d: V) -> FiniteSet[V] {
    fs_insert(fs_insert(fs_insert(fs_insert(FiniteSet.empty[V], a), b), c), d)
}

/// The witness set holds all four vertices.
theorem claw_witness_set_contains[V](a: V, b: V, c: V, d: V) {
    claw_witness_set(a, b, c, d).contains(a) and claw_witness_set(a, b, c, d).contains(b)
        and claw_witness_set(a, b, c, d).contains(c)
        and claw_witness_set(a, b, c, d).contains(d)
} by {
    fs_insert_contains_item(FiniteSet.empty[V], a)
    fs_insert(FiniteSet.empty[V], a).contains(a)
    fs_insert_contains_of_contains(fs_insert(FiniteSet.empty[V], a), b, a)
    fs_insert(fs_insert(FiniteSet.empty[V], a), b).contains(a)
    fs_insert_contains_item(fs_insert(FiniteSet.empty[V], a), b)
    fs_insert(fs_insert(FiniteSet.empty[V], a), b).contains(b)
    fs_insert_contains_of_contains(
        fs_insert(fs_insert(FiniteSet.empty[V], a), b), c, a)
    fs_insert(fs_insert(fs_insert(FiniteSet.empty[V], a), b), c).contains(a)
    fs_insert_contains_of_contains(
        fs_insert(fs_insert(FiniteSet.empty[V], a), b), c, b)
    fs_insert(fs_insert(fs_insert(FiniteSet.empty[V], a), b), c).contains(b)
    fs_insert_contains_item(fs_insert(fs_insert(FiniteSet.empty[V], a), b), c)
    fs_insert(fs_insert(fs_insert(FiniteSet.empty[V], a), b), c).contains(c)
    fs_insert_contains_of_contains(
        fs_insert(fs_insert(fs_insert(FiniteSet.empty[V], a), b), c), d, a)
    claw_witness_set(a, b, c, d).contains(a)
    fs_insert_contains_of_contains(
        fs_insert(fs_insert(fs_insert(FiniteSet.empty[V], a), b), c), d, b)
    claw_witness_set(a, b, c, d).contains(b)
    fs_insert_contains_of_contains(
        fs_insert(fs_insert(fs_insert(FiniteSet.empty[V], a), b), c), d, c)
    claw_witness_set(a, b, c, d).contains(c)
    fs_insert_contains_item(
        fs_insert(fs_insert(fs_insert(FiniteSet.empty[V], a), b), c), d)
    claw_witness_set(a, b, c, d).contains(d)
    (claw_witness_set(a, b, c, d).contains(a) and claw_witness_set(a, b, c, d).contains(b)
        and claw_witness_set(a, b, c, d).contains(c)
        and claw_witness_set(a, b, c, d).contains(d))
}

/// An induced claw gives a center with three independent neighbors.
///
/// The four images of the embedding are the configuration: the image of the center is adjacent
/// to the other three, which are pairwise distinct because the embedding is injective and
/// pairwise non-adjacent because the embedding reflects adjacency as well as preserving it.
theorem triple_of_contains_induced_claw[V](g: SimpleGraph[V]) {
    contains_induced(g, claw4_graph) implies exists(s: FiniteSet[V], a: V) {
        s.contains(a) and has_independent_neighbor_triple(g, s, a)
    }
} by {
    if contains_induced(g, claw4_graph) {
        contains_induced_witness(g, claw4_graph)
        exists(f: Fin[Nat.4] -> V) { is_graph_embedding(claw4_graph, g, f) }
        let (m: Fin[Nat.4] -> V) satisfy {
            is_graph_embedding(claw4_graph, g, m)
        }
        claw4_labels
        (Nat.0 < Nat.4 and Nat.1 < Nat.4 and Nat.2 < Nat.4 and Nat.3 < Nat.4
            and Nat.0 != Nat.1 and Nat.0 != Nat.2 and Nat.0 != Nat.3
            and Nat.1 != Nat.2 and Nat.1 != Nat.3 and Nat.2 != Nat.3)
        claw4_witness(Nat.0)
        exists(i: Fin[Nat.4]) { i.value = Nat.0 }
        let (v0: Fin[Nat.4]) satisfy {
            v0.value = Nat.0
        }
        claw4_witness(Nat.1)
        exists(i: Fin[Nat.4]) { i.value = Nat.1 }
        let (v1: Fin[Nat.4]) satisfy {
            v1.value = Nat.1
        }
        claw4_witness(Nat.2)
        exists(i: Fin[Nat.4]) { i.value = Nat.2 }
        let (v2: Fin[Nat.4]) satisfy {
            v2.value = Nat.2
        }
        claw4_witness(Nat.3)
        exists(i: Fin[Nat.4]) { i.value = Nat.3 }
        let (v3: Fin[Nat.4]) satisfy {
            v3.value = Nat.3
        }
        claw4_graph_adj_iff(v0, v1)
        (claw4_graph.adj(v0, v1) = ((v0.value = Nat.0 and v1.value != Nat.0)
            or (v1.value = Nat.0 and v0.value != Nat.0)))
        claw4_graph.adj(v0, v1)
        graph_embedding_adj_forward(claw4_graph, g, m, v0, v1)
        g.adj(m(v0), m(v1))
        claw4_graph_adj_iff(v0, v2)
        (claw4_graph.adj(v0, v2) = ((v0.value = Nat.0 and v2.value != Nat.0)
            or (v2.value = Nat.0 and v0.value != Nat.0)))
        claw4_graph.adj(v0, v2)
        graph_embedding_adj_forward(claw4_graph, g, m, v0, v2)
        g.adj(m(v0), m(v2))
        claw4_graph_adj_iff(v0, v3)
        (claw4_graph.adj(v0, v3) = ((v0.value = Nat.0 and v3.value != Nat.0)
            or (v3.value = Nat.0 and v0.value != Nat.0)))
        claw4_graph.adj(v0, v3)
        graph_embedding_adj_forward(claw4_graph, g, m, v0, v3)
        g.adj(m(v0), m(v3))
        claw4_graph_adj_iff(v1, v2)
        (claw4_graph.adj(v1, v2) = ((v1.value = Nat.0 and v2.value != Nat.0)
            or (v2.value = Nat.0 and v1.value != Nat.0)))
        not claw4_graph.adj(v1, v2)
        if g.adj(m(v1), m(v2)) {
            graph_embedding_adj_backward(claw4_graph, g, m, v1, v2)
            claw4_graph.adj(v1, v2)
            false
        }
        not g.adj(m(v1), m(v2))
        claw4_graph_adj_iff(v1, v3)
        (claw4_graph.adj(v1, v3) = ((v1.value = Nat.0 and v3.value != Nat.0)
            or (v3.value = Nat.0 and v1.value != Nat.0)))
        not claw4_graph.adj(v1, v3)
        if g.adj(m(v1), m(v3)) {
            graph_embedding_adj_backward(claw4_graph, g, m, v1, v3)
            claw4_graph.adj(v1, v3)
            false
        }
        not g.adj(m(v1), m(v3))
        claw4_graph_adj_iff(v2, v3)
        (claw4_graph.adj(v2, v3) = ((v2.value = Nat.0 and v3.value != Nat.0)
            or (v3.value = Nat.0 and v2.value != Nat.0)))
        not claw4_graph.adj(v2, v3)
        if g.adj(m(v2), m(v3)) {
            graph_embedding_adj_backward(claw4_graph, g, m, v2, v3)
            claw4_graph.adj(v2, v3)
            false
        }
        not g.adj(m(v2), m(v3))
        graph_embedding_is_injective(claw4_graph, g, m)
        is_injective_fn(m)
        if m(v1) = m(v2) {
            injective_fn_eq(m, v1, v2)
            v1 = v2
            (v1.value = v2.value)
            false
        }
        m(v1) != m(v2)
        if m(v1) = m(v3) {
            injective_fn_eq(m, v1, v3)
            v1 = v3
            (v1.value = v3.value)
            false
        }
        m(v1) != m(v3)
        if m(v2) = m(v3) {
            injective_fn_eq(m, v2, v3)
            v2 = v3
            (v2.value = v3.value)
            false
        }
        m(v2) != m(v3)
        claw_witness_set_contains(m(v0), m(v1), m(v2), m(v3))
        (claw_witness_set(m(v0), m(v1), m(v2), m(v3)).contains(m(v0))
            and claw_witness_set(m(v0), m(v1), m(v2), m(v3)).contains(m(v1))
            and claw_witness_set(m(v0), m(v1), m(v2), m(v3)).contains(m(v2))
            and claw_witness_set(m(v0), m(v1), m(v2), m(v3)).contains(m(v3)))
        neighborhood_contains_eq(g, claw_witness_set(m(v0), m(v1), m(v2), m(v3)),
            m(v0), m(v1))
        (neighborhood(g, claw_witness_set(m(v0), m(v1), m(v2), m(v3)), m(v0)).contains(m(v1))
            = (claw_witness_set(m(v0), m(v1), m(v2), m(v3)).contains(m(v1))
                and g.adj(m(v0), m(v1))))
        neighborhood(g, claw_witness_set(m(v0), m(v1), m(v2), m(v3)), m(v0)).contains(m(v1))
        neighborhood_contains_eq(g, claw_witness_set(m(v0), m(v1), m(v2), m(v3)),
            m(v0), m(v2))
        (neighborhood(g, claw_witness_set(m(v0), m(v1), m(v2), m(v3)), m(v0)).contains(m(v2))
            = (claw_witness_set(m(v0), m(v1), m(v2), m(v3)).contains(m(v2))
                and g.adj(m(v0), m(v2))))
        neighborhood(g, claw_witness_set(m(v0), m(v1), m(v2), m(v3)), m(v0)).contains(m(v2))
        neighborhood_contains_eq(g, claw_witness_set(m(v0), m(v1), m(v2), m(v3)),
            m(v0), m(v3))
        (neighborhood(g, claw_witness_set(m(v0), m(v1), m(v2), m(v3)), m(v0)).contains(m(v3))
            = (claw_witness_set(m(v0), m(v1), m(v2), m(v3)).contains(m(v3))
                and g.adj(m(v0), m(v3))))
        neighborhood(g, claw_witness_set(m(v0), m(v1), m(v2), m(v3)), m(v0)).contains(m(v3))
        has_independent_neighbor_triple_intro(g,
            claw_witness_set(m(v0), m(v1), m(v2), m(v3)), m(v0), m(v1), m(v2), m(v3))
        has_independent_neighbor_triple(g,
            claw_witness_set(m(v0), m(v1), m(v2), m(v3)), m(v0))
        claw_witness_set(m(v0), m(v1), m(v2), m(v3)).contains(m(v0))
        exists(s: FiniteSet[V], a: V) {
            s.contains(a) and has_independent_neighbor_triple(g, s, a)
        }
    }
}

/// A graph claw-free on every vertex set is free of the claw.
theorem free_of_claw_of_is_claw_free[V](g: SimpleGraph[V]) {
    (forall(s: FiniteSet[V]) { is_claw_free(g, s) }) implies is_free_of(g, claw4_graph)
} by {
    if forall(s: FiniteSet[V]) { is_claw_free(g, s) } {
        if contains_induced(g, claw4_graph) {
            triple_of_contains_induced_claw(g)
            exists(s: FiniteSet[V], a: V) {
                s.contains(a) and has_independent_neighbor_triple(g, s, a)
            }
            let (t: FiniteSet[V], w: V) satisfy {
                t.contains(w) and has_independent_neighbor_triple(g, t, w)
            }
            is_claw_free(g, t)
            is_claw_free_apply(g, t, w)
            (is_claw_free(g, t) and t.contains(w)
                implies not has_independent_neighbor_triple(g, t, w))
            not has_independent_neighbor_triple(g, t, w)
            false
        }
        not contains_induced(g, claw4_graph)
        is_free_of_intro(g, claw4_graph)
        is_free_of(g, claw4_graph)
    }
}

/// Freedom from the four-vertex claw and claw-freeness on every vertex set agree.
///
/// The equivalence the two readings were built for. One is a statement about an embedding of a
/// four-vertex graph, the other about neighborhoods inside an ambient set, and they say the
/// same thing.
theorem free_of_claw_iff_claw_free[V](g: SimpleGraph[V]) {
    is_free_of(g, claw4_graph) = forall(s: FiniteSet[V]) { is_claw_free(g, s) }
} by {
    if is_free_of(g, claw4_graph) {
        forall(s: FiniteSet[V]) {
            is_claw_free_of_free_of_claw(g, s)
            is_claw_free(g, s)
        }
    }
    if forall(s: FiniteSet[V]) { is_claw_free(g, s) } {
        free_of_claw_of_is_claw_free(g)
        is_free_of(g, claw4_graph)
    }
    is_free_of(g, claw4_graph) = forall(s: FiniteSet[V]) { is_claw_free(g, s) }
}
