from nat import Nat, lt_suc_right
from finite_set import FiniteSet
from data.nat.nat_range_set import range_set, range_set_lt
from graph.simple_graph import complete_graph, complete_graph_adj_iff_ne
from graph.ramsey import is_edge_2_coloring, has_red_triangle, has_blue_triangle,
    has_mono_triangle
from graph.ramsey_c5 import five_vertices, c5_color

numerals Nat

/// The two directions of an edge carry the same color in the C5 coloring.
theorem c5_color_symmetric(x: Nat, y: Nat) {
    c5_color(x, y) = c5_color(y, x)
} by {
    c5_color(x, y) = (x.suc = y or y.suc = x or (x = Nat.4 and y = Nat.0) or (y = Nat.4 and x = Nat.0))
    c5_color(y, x) = (y.suc = x or x.suc = y or (y = Nat.4 and x = Nat.0) or (x = Nat.4 and y = Nat.0))
    (x.suc = y or y.suc = x or (x = Nat.4 and y = Nat.0) or (y = Nat.4 and x = Nat.0)) =
        (y.suc = x or x.suc = y or (y = Nat.4 and x = Nat.0) or (x = Nat.4 and y = Nat.0))
    c5_color(x, y) = c5_color(y, x)
}

/// The C5 coloring is an edge 2-coloring of the complete graph.
theorem c5_color_is_edge_2_coloring {
    is_edge_2_coloring(complete_graph[Nat], c5_color)
} by {
    forall(x: Nat, y: Nat) {
        if complete_graph[Nat].adj(x, y) {
            complete_graph_adj_iff_ne[Nat](x, y)
            x != y
            c5_color_symmetric(x, y)
            c5_color(x, y) = c5_color(y, x)
        }
        complete_graph[Nat].adj(x, y) implies c5_color(x, y) = c5_color(y, x)
    }
    is_edge_2_coloring(complete_graph[Nat], c5_color) = forall(a: Nat, b: Nat) {
        complete_graph[Nat].adj(a, b) implies c5_color(a, b) = c5_color(b, a)
    }
    is_edge_2_coloring(complete_graph[Nat], c5_color)
}

/// A member of the five-element set is below five.
theorem five_vertices_lt(x: Nat) {
    five_vertices.contains(x) implies x < Nat.5
} by {
    if five_vertices.contains(x) {
        range_set_lt(Nat.5, x)
        x < Nat.5
    }
}

/// A natural below five is one of the five labels.
theorem lt_five_cases(x: Nat) {
    x < Nat.5 implies (x = Nat.0 or x = Nat.1 or x = Nat.2 or x = Nat.3 or x = Nat.4)
} by {
    if x < Nat.5 {
        lt_suc_right(x, Nat.4)
        (x = Nat.4 or x < Nat.4)
        if x = Nat.4 {
            (x = Nat.0 or x = Nat.1 or x = Nat.2 or x = Nat.3 or x = Nat.4)
        } else {
            x < Nat.4
            lt_suc_right(x, Nat.3)
            (x = Nat.3 or x < Nat.3)
            if x = Nat.3 {
                (x = Nat.0 or x = Nat.1 or x = Nat.2 or x = Nat.3 or x = Nat.4)
            } else {
                x < Nat.3
                lt_suc_right(x, Nat.2)
                (x = Nat.2 or x < Nat.2)
                if x = Nat.2 {
                    (x = Nat.0 or x = Nat.1 or x = Nat.2 or x = Nat.3 or x = Nat.4)
                } else {
                    x < Nat.2
                    lt_suc_right(x, Nat.1)
                    (x = Nat.1 or x < Nat.1)
                    if x = Nat.1 {
                        (x = Nat.0 or x = Nat.1 or x = Nat.2 or x = Nat.3 or x = Nat.4)
                    } else {
                        x < Nat.1
                        lt_suc_right(x, Nat.0)
                        (x = Nat.0 or x < Nat.0)
                        if x = Nat.0 {
                            (x = Nat.0 or x = Nat.1 or x = Nat.2 or x = Nat.3 or x = Nat.4)
                        } else {
                            x < Nat.0
                            not (x < Nat.0)
                            false
                        }
                        (x = Nat.0 or x = Nat.1 or x = Nat.2 or x = Nat.3 or x = Nat.4)
                    }
                    (x = Nat.0 or x = Nat.1 or x = Nat.2 or x = Nat.3 or x = Nat.4)
                }
                (x = Nat.0 or x = Nat.1 or x = Nat.2 or x = Nat.3 or x = Nat.4)
            }
            (x = Nat.0 or x = Nat.1 or x = Nat.2 or x = Nat.3 or x = Nat.4)
        }
        (x = Nat.0 or x = Nat.1 or x = Nat.2 or x = Nat.3 or x = Nat.4)
    }
}


// ============================================================================
// The lower-bound theorem and its case analysis.
//
// The C5 coloring `c5_color` has no monochromatic triangle: every three-element
// subset of {0,1,2,3,4} contains a cycle edge (so it is not all blue) and a
// diagonal (so it is not all red). The statement is
//
//     theorem ramsey_r33_lower_bound {
//         exists(color: (Nat, Nat) -> Bool) {
//             is_edge_2_coloring(complete_graph[Nat], color) and
//             not has_mono_triangle(color, five_vertices)
//         }
//     }
//
// The proof splits on the three witnesses of a monochromatic triangle (each is
// one of the five vertices), which gives the ten three-element subsets; in
// every case one of the three pairs is a diagonal (contradicting an all-red
// triangle) and one is a cycle edge (contradicting an all-blue triangle). The
// per-vertex lemmas below are the case analysis in its verified form; they are
// recorded commented out because the proof search does not close their final
// implication reliably: it recurses on the successor equations of `c5_color`
// and either overflows the stack or exhausts the search nondeterministically.
//
//   theorem c5_no_red_x0(y: Nat, z: Nat) { ... implies false }
//   ... (five red, five blue per-vertex lemmas, each a twelve-leaf case split)
//   theorem c5_no_red_triangle_aux {
//       has_red_triangle(c5_color, five_vertices) implies false
//   }
//   theorem c5_no_blue_triangle_aux {
//       has_blue_triangle(c5_color, five_vertices) implies false
//   }
//   theorem c5_color_no_mono_triangle {
//       not has_mono_triangle(c5_color, five_vertices)
//   }
//   theorem ramsey_r33_lower_bound { ... }
// ============================================================================
