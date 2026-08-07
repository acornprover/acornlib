from nat import Nat, lte_trans
from finite_set import FiniteSet, finite_set_subset_refl, finite_set_subset_trans,
    finite_set_subset_contains
from data.finite.finite_set_card import fs_card
from data.finite.finite_set_subset_intro import fs_subset_eq_intro
from data.finite.finite_set_membership import finite_set_eq_of_contains_eq
from graph.simple_graph import SimpleGraph
from graph.simple_graph_zero_forcing_rule import derived_set, derived_set_subset,
    derived_set_grows, derived_set_contains_eq, derived_set_eq_of_closed,
    is_forcing_closed, ambient_is_forcing_closed

numerals Nat

/// The blue set after `n` rounds of forcing, starting from `b`.
///
/// Each round colours every currently forceable vertex at once. Applying the rule one vertex
/// at a time reaches the same final colouring, so this loses nothing and makes each round a
/// function of the previous colouring alone.
define forcing_iterate[V](
    g: SimpleGraph[V], s: FiniteSet[V], b: FiniteSet[V], n: Nat
) -> FiniteSet[V] {
    match n {
        Nat.zero {
            b
        }
        Nat.suc(k) {
            derived_set(g, s, forcing_iterate(g, s, b, k))
        }
    }
}

/// No rounds leaves the colouring unchanged.
theorem forcing_iterate_zero[V](g: SimpleGraph[V], s: FiniteSet[V], b: FiniteSet[V]) {
    forcing_iterate(g, s, b, Nat.0) = b
}

/// One more round is one application of the rule.
theorem forcing_iterate_suc[V](
    g: SimpleGraph[V], s: FiniteSet[V], b: FiniteSet[V], k: Nat
) {
    forcing_iterate(g, s, b, k.suc) = derived_set(g, s, forcing_iterate(g, s, b, k))
}

/// Every stage of the iteration stays inside the ambient set.
theorem forcing_iterate_subset[V](
    g: SimpleGraph[V], s: FiniteSet[V], b: FiniteSet[V], n: Nat
) {
    b.subset_eq(s) implies forcing_iterate(g, s, b, n).subset_eq(s)
} by {
    if b.subset_eq(s) {
        define p(x: Nat) -> Bool {
            forcing_iterate(g, s, b, x).subset_eq(s)
        }
        forcing_iterate(g, s, b, Nat.0) = b
        forcing_iterate(g, s, b, Nat.0).subset_eq(s)
        p(Nat.0)
        forall(k: Nat) {
            if p(k) {
                forcing_iterate(g, s, b, k).subset_eq(s)
                forcing_iterate_suc(g, s, b, k)
                forcing_iterate(g, s, b, k.suc) = derived_set(g, s, forcing_iterate(g, s, b, k))
                derived_set_subset(g, s, forcing_iterate(g, s, b, k))
                derived_set(g, s, forcing_iterate(g, s, b, k)).subset_eq(s)
                forcing_iterate(g, s, b, k.suc).subset_eq(s)
                p(k.suc)
            }
            (p(k) implies p(k.suc))
        }
        p(Nat.0) and forall(k: Nat) {
            p(k) implies p(k.suc)
        }
        Nat.induction(p)
        p(n)
        forcing_iterate(g, s, b, n).subset_eq(s)
    }
}

/// One more round only adds vertices.
theorem forcing_iterate_step_grows[V](
    g: SimpleGraph[V], s: FiniteSet[V], b: FiniteSet[V], n: Nat
) {
    b.subset_eq(s)
        implies forcing_iterate(g, s, b, n).subset_eq(forcing_iterate(g, s, b, n.suc))
} by {
    if b.subset_eq(s) {
        forcing_iterate_subset(g, s, b, n)
        forcing_iterate(g, s, b, n).subset_eq(s)
        derived_set_grows(g, s, forcing_iterate(g, s, b, n))
        forcing_iterate(g, s, b, n).subset_eq(derived_set(g, s, forcing_iterate(g, s, b, n)))
        forcing_iterate_suc(g, s, b, n)
        forcing_iterate(g, s, b, n.suc) = derived_set(g, s, forcing_iterate(g, s, b, n))
        forcing_iterate(g, s, b, n).subset_eq(forcing_iterate(g, s, b, n.suc))
    }
}

/// The starting colouring survives every round.
theorem forcing_iterate_contains_start[V](
    g: SimpleGraph[V], s: FiniteSet[V], b: FiniteSet[V], n: Nat
) {
    b.subset_eq(s) implies b.subset_eq(forcing_iterate(g, s, b, n))
} by {
    if b.subset_eq(s) {
        define q(x: Nat) -> Bool {
            b.subset_eq(forcing_iterate(g, s, b, x))
        }
        forcing_iterate(g, s, b, Nat.0) = b
        finite_set_subset_refl(b)
        b.subset_eq(forcing_iterate(g, s, b, Nat.0))
        q(Nat.0)
        forall(k: Nat) {
            if q(k) {
                b.subset_eq(forcing_iterate(g, s, b, k))
                forcing_iterate_step_grows(g, s, b, k)
                forcing_iterate(g, s, b, k).subset_eq(forcing_iterate(g, s, b, k.suc))
                finite_set_subset_trans(b, forcing_iterate(g, s, b, k),
                    forcing_iterate(g, s, b, k.suc))
                b.subset_eq(forcing_iterate(g, s, b, k.suc))
                q(k.suc)
            }
            (q(k) implies q(k.suc))
        }
        q(Nat.0) and forall(k: Nat) {
            q(k) implies q(k.suc)
        }
        Nat.induction(q)
        q(n)
        b.subset_eq(forcing_iterate(g, s, b, n))
    }
}

/// Once a round is closed, every later round repeats it.
///
/// This is what makes reaching the ambient set a stable property rather than one that could
/// be lost by iterating further.
theorem forcing_iterate_stationary[V](
    g: SimpleGraph[V], s: FiniteSet[V], b: FiniteSet[V], n: Nat, m: Nat
) {
    b.subset_eq(s) and is_forcing_closed(g, s, forcing_iterate(g, s, b, n))
        implies forcing_iterate(g, s, b, n.suc) = forcing_iterate(g, s, b, n)
} by {
    if b.subset_eq(s) and is_forcing_closed(g, s, forcing_iterate(g, s, b, n)) {
        forcing_iterate_subset(g, s, b, n)
        forcing_iterate(g, s, b, n).subset_eq(s)
        forall(v: V) {
            derived_set_eq_of_closed(g, s, forcing_iterate(g, s, b, n), v)
            derived_set(g, s, forcing_iterate(g, s, b, n)).contains(v) =
                forcing_iterate(g, s, b, n).contains(v)
        }
        finite_set_eq_of_contains_eq(derived_set(g, s, forcing_iterate(g, s, b, n)),
            forcing_iterate(g, s, b, n))
        derived_set(g, s, forcing_iterate(g, s, b, n)) = forcing_iterate(g, s, b, n)
        forcing_iterate_suc(g, s, b, n)
        forcing_iterate(g, s, b, n.suc) = forcing_iterate(g, s, b, n)
    }
}

/// True if repeated forcing from `b` eventually colours all of `s`.
///
/// The standard definition of a zero forcing set. No bound on the number of rounds is needed
/// here: the ambient set is finite, so an iteration that ever reaches it does so at some
/// stage, and that stage is what the existential names.
define is_zero_forcing_set[V](
    g: SimpleGraph[V], s: FiniteSet[V], b: FiniteSet[V]
) -> Bool {
    b.subset_eq(s) and exists(n: Nat) {
        forcing_iterate(g, s, b, n) = s
    }
}

/// A zero forcing set is a subset that reaches the ambient set.
theorem is_zero_forcing_set_apply[V](
    g: SimpleGraph[V], s: FiniteSet[V], b: FiniteSet[V]
) {
    is_zero_forcing_set(g, s, b)
        implies b.subset_eq(s) and exists(n: Nat) { forcing_iterate(g, s, b, n) = s }
} by {
    if is_zero_forcing_set(g, s, b) {
        is_zero_forcing_set(g, s, b) = (b.subset_eq(s) and exists(m: Nat) {
            forcing_iterate(g, s, b, m) = s
        })
        b.subset_eq(s) and exists(m: Nat) { forcing_iterate(g, s, b, m) = s }
    }
}

/// A stage reaching the ambient set makes the starting set a zero forcing set.
theorem is_zero_forcing_set_intro[V](
    g: SimpleGraph[V], s: FiniteSet[V], b: FiniteSet[V], n: Nat
) {
    b.subset_eq(s) and forcing_iterate(g, s, b, n) = s implies is_zero_forcing_set(g, s, b)
} by {
    if b.subset_eq(s) and forcing_iterate(g, s, b, n) = s {
        exists(m: Nat) {
            forcing_iterate(g, s, b, m) = s
        }
        is_zero_forcing_set(g, s, b) = (b.subset_eq(s) and exists(k: Nat) {
            forcing_iterate(g, s, b, k) = s
        })
        is_zero_forcing_set(g, s, b)
    }
}

/// The whole vertex set is a zero forcing set.
///
/// It is already entirely blue, so zero rounds suffice. This is what makes the zero forcing
/// number well defined.
theorem ambient_is_zero_forcing_set[V](g: SimpleGraph[V], s: FiniteSet[V]) {
    is_zero_forcing_set(g, s, s)
} by {
    finite_set_subset_refl(s)
    s.subset_eq(s)
    forcing_iterate(g, s, s, Nat.0) = s
    is_zero_forcing_set_intro(g, s, s, Nat.0)
    is_zero_forcing_set(g, s, s)
}
