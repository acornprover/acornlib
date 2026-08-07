from nat import Nat
from finite_set import FiniteSet
from data.finite.finite_set_filter import finite_set_filter, finite_set_filter_contains_eq,
    finite_set_filter_subset
from graph.simple_graph import SimpleGraph
from graph.simple_graph_bipartite import is_bipartition, is_bipartition_apply,
    is_bipartition_intro, is_bipartition_same_side_not_adj
from graph.simple_graph_independent import is_independent_in, is_independent_in_intro

numerals Nat

/// The complement of a two-colouring.
define opposite_part[V](part: V -> Bool) -> (V -> Bool) {
    function(v: V) {
        not part(v)
    }
}

/// A vertex lies on the opposite side exactly when it is not on the given one.
theorem opposite_part_eq[V](part: V -> Bool, v: V) {
    opposite_part(part)(v) = not part(v)
}

/// The vertices of `s` coloured by `part`.
define bipartite_side[V](s: FiniteSet[V], part: V -> Bool) -> FiniteSet[V] {
    finite_set_filter(s, part)
}

/// A vertex is on a side exactly when it is in the ambient set and has that colour.
theorem bipartite_side_contains_eq[V](s: FiniteSet[V], part: V -> Bool, v: V) {
    bipartite_side(s, part).contains(v) = (s.contains(v) and part(v))
} by {
    finite_set_filter_contains_eq(s, part, v)
}

/// A side sits inside the ambient set.
theorem bipartite_side_subset[V](s: FiniteSet[V], part: V -> Bool) {
    bipartite_side(s, part).subset_eq(s)
} by {
    finite_set_filter_subset(s, part)
}

/// Each side of a bipartition is an independent set.
///
/// Two vertices of the same colour are never adjacent, which is the defining
/// property of the colouring read one side at a time.
theorem bipartite_side_independent[V](
    g: SimpleGraph[V], s: FiniteSet[V], part: V -> Bool
) {
    is_bipartition(g, s, part) implies is_independent_in(g, bipartite_side(s, part))
} by {
    if is_bipartition(g, s, part) {
        forall(x: V, y: V) {
            if bipartite_side(s, part).contains(x) and bipartite_side(s, part).contains(y) {
                bipartite_side_contains_eq(s, part, x)
                s.contains(x) and part(x)
                bipartite_side_contains_eq(s, part, y)
                s.contains(y) and part(y)
                part(x) = part(y)
                is_bipartition_same_side_not_adj(g, s, part, x, y)
                not g.adj(x, y)
            }
            bipartite_side(s, part).contains(x) and bipartite_side(s, part).contains(y) implies not g.adj(x, y)
        }
        is_independent_in_intro(g, bipartite_side(s, part))
        is_independent_in(g, bipartite_side(s, part))
    }
}

/// A bipartition of a colouring is a bipartition of its complement.
theorem is_bipartition_opposite[V](
    g: SimpleGraph[V], s: FiniteSet[V], part: V -> Bool
) {
    is_bipartition(g, s, part) implies is_bipartition(g, s, opposite_part(part))
} by {
    if is_bipartition(g, s, part) {
        forall(x: V, y: V) {
            if s.contains(x) and s.contains(y) and g.adj(x, y) {
                is_bipartition_apply(g, s, part, x, y)
                part(x) != part(y)
                opposite_part_eq(part, x)
                opposite_part(part)(x) = not part(x)
                opposite_part_eq(part, y)
                opposite_part(part)(y) = not part(y)
                if part(x) {
                    not part(y)
                    not opposite_part(part)(x)
                    opposite_part(part)(y)
                    opposite_part(part)(x) != opposite_part(part)(y)
                }
                if not part(x) {
                    part(y)
                    opposite_part(part)(x)
                    not opposite_part(part)(y)
                    opposite_part(part)(x) != opposite_part(part)(y)
                }
                opposite_part(part)(x) != opposite_part(part)(y)
            }
            s.contains(x) and s.contains(y) and g.adj(x, y) implies opposite_part(part)(x) != opposite_part(part)(y)
        }
        is_bipartition_intro(g, s, opposite_part(part))
        is_bipartition(g, s, opposite_part(part))
    }
}

/// The other side of a bipartition is also independent.
theorem bipartite_other_side_independent[V](
    g: SimpleGraph[V], s: FiniteSet[V], part: V -> Bool
) {
    is_bipartition(g, s, part) implies is_independent_in(g, bipartite_side(s, opposite_part(part)))
} by {
    if is_bipartition(g, s, part) {
        is_bipartition_opposite(g, s, part)
        is_bipartition(g, s, opposite_part(part))
        bipartite_side_independent(g, s, opposite_part(part))
        is_independent_in(g, bipartite_side(s, opposite_part(part)))
    }
}

/// Every vertex of the ambient set lies on exactly one side.
theorem bipartite_sides_cover[V](s: FiniteSet[V], part: V -> Bool, v: V) {
    s.contains(v) implies bipartite_side(s, part).contains(v) or bipartite_side(s, opposite_part(part)).contains(v)
} by {
    if s.contains(v) {
        if part(v) {
            bipartite_side_contains_eq(s, part, v)
            bipartite_side(s, part).contains(v)
        }
        if not part(v) {
            opposite_part_eq(part, v)
            opposite_part(part)(v)
            bipartite_side_contains_eq(s, opposite_part(part), v)
            bipartite_side(s, opposite_part(part)).contains(v)
        }
        bipartite_side(s, part).contains(v) or bipartite_side(s, opposite_part(part)).contains(v)
    }
}

/// No vertex lies on both sides.
theorem bipartite_sides_disjoint[V](s: FiniteSet[V], part: V -> Bool, v: V) {
    not (bipartite_side(s, part).contains(v) and bipartite_side(s, opposite_part(part)).contains(v))
} by {
    if bipartite_side(s, part).contains(v) and bipartite_side(s, opposite_part(part)).contains(v) {
        bipartite_side_contains_eq(s, part, v)
        part(v)
        bipartite_side_contains_eq(s, opposite_part(part), v)
        opposite_part(part)(v)
        opposite_part_eq(part, v)
        not part(v)
        false
    }
    not (bipartite_side(s, part).contains(v) and bipartite_side(s, opposite_part(part)).contains(v))
}
