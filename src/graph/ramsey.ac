from nat import Nat, lt_and_lte
from finite_set import FiniteSet
from data.basic.logic import exists_intro
from data.nat.nat_range_set import range_set, range_set_contains, range_set_contains_eq
from graph.simple_graph import SimpleGraph, complete_graph, complete_graph_adj_iff_ne
from graph.simple_graph_degree import neighborhood, neighborhood_contains_eq

numerals Nat

/// The six vertices `{0, ..., 5}` on which Ramsey's theorem is stated.
let six_vertices: FiniteSet[Nat] = range_set(Nat.6)

/// The five vertices `{0, ..., 4}` on which the lower bound lives.
let five_vertices: FiniteSet[Nat] = range_set(Nat.5)

/// The labels of the six vertices are ordered, and each is below the bound.
///
/// Comparisons between numerals are not free, so the ladder is stated once and read off.
theorem six_labels_lt {
    Nat.0 < Nat.1 and Nat.1 < Nat.2 and Nat.2 < Nat.3 and Nat.3 < Nat.4 and Nat.4 < Nat.5
    and Nat.5 < Nat.6
    and Nat.0 < Nat.6 and Nat.1 < Nat.6 and Nat.2 < Nat.6 and Nat.3 < Nat.6 and Nat.4 < Nat.6
} by {
    Nat.0.suc = Nat.1
    Nat.0 < Nat.0.suc
    Nat.0 < Nat.1
    Nat.1.suc = Nat.2
    Nat.1 < Nat.1.suc
    Nat.1 < Nat.2
    Nat.2.suc = Nat.3
    Nat.2 < Nat.2.suc
    Nat.2 < Nat.3
    Nat.3.suc = Nat.4
    Nat.3 < Nat.3.suc
    Nat.3 < Nat.4
    Nat.4.suc = Nat.5
    Nat.4 < Nat.4.suc
    Nat.4 < Nat.5
    Nat.5.suc = Nat.6
    Nat.5 < Nat.5.suc
    Nat.5 < Nat.6
    Nat.5 <= Nat.6
    lt_and_lte(Nat.4, Nat.5, Nat.6)
    Nat.4 < Nat.6
    Nat.4 <= Nat.6
    lt_and_lte(Nat.3, Nat.4, Nat.6)
    Nat.3 < Nat.6
    Nat.3 <= Nat.6
    lt_and_lte(Nat.2, Nat.3, Nat.6)
    Nat.2 < Nat.6
    Nat.2 <= Nat.6
    lt_and_lte(Nat.1, Nat.2, Nat.6)
    Nat.1 < Nat.6
    Nat.1 <= Nat.6
    lt_and_lte(Nat.0, Nat.1, Nat.6)
    Nat.0 < Nat.6
}

/// The six labels are pairwise distinct, and none is zero.
theorem six_labels_distinct {
    Nat.0 != Nat.1 and Nat.0 != Nat.2 and Nat.0 != Nat.3 and Nat.0 != Nat.4 and Nat.0 != Nat.5
    and Nat.1 != Nat.2 and Nat.1 != Nat.3 and Nat.1 != Nat.4 and Nat.1 != Nat.5
    and Nat.2 != Nat.3 and Nat.2 != Nat.4 and Nat.2 != Nat.5
    and Nat.3 != Nat.4 and Nat.3 != Nat.5
    and Nat.4 != Nat.5
} by {
    six_labels_lt
    (Nat.0 < Nat.1 and Nat.1 < Nat.2 and Nat.2 < Nat.3 and Nat.3 < Nat.4 and Nat.4 < Nat.5
        and Nat.5 < Nat.6)
    Nat.0 < Nat.1
    Nat.0 != Nat.1
    Nat.0 < Nat.2
    Nat.0 != Nat.2
    Nat.0 < Nat.3
    Nat.0 != Nat.3
    Nat.0 < Nat.4
    Nat.0 != Nat.4
    Nat.0 < Nat.5
    Nat.0 != Nat.5
    Nat.1 < Nat.2
    Nat.1 != Nat.2
    Nat.1 < Nat.3
    Nat.1 != Nat.3
    Nat.1 < Nat.4
    Nat.1 != Nat.4
    Nat.1 < Nat.5
    Nat.1 != Nat.5
    Nat.2 < Nat.3
    Nat.2 != Nat.3
    Nat.2 < Nat.4
    Nat.2 != Nat.4
    Nat.2 < Nat.5
    Nat.2 != Nat.5
    Nat.3 < Nat.4
    Nat.3 != Nat.4
    Nat.3 < Nat.5
    Nat.3 != Nat.5
    Nat.4 < Nat.5
    Nat.4 != Nat.5
}

/// A natural below six is a vertex of the six-element set.
theorem six_vertices_contains(x: Nat) {
    x < Nat.6 implies six_vertices.contains(x)
} by {
    if x < Nat.6 {
        range_set_contains(Nat.6, x)
        range_set(Nat.6).contains(x)
        six_vertices.contains(x)
    }
}

/// A natural below five is a vertex of the five-element set.
theorem five_vertices_contains(x: Nat) {
    x < Nat.5 implies five_vertices.contains(x)
} by {
    if x < Nat.5 {
        range_set_contains(Nat.5, x)
        range_set(Nat.5).contains(x)
        five_vertices.contains(x)
    }
}

/// A vertex of the six-element set other than zero is a neighbour of zero in the complete graph.
theorem six_neighbor_of_zero(x: Nat) {
    six_vertices.contains(x) and x != Nat.0 implies
        neighborhood(complete_graph[Nat], six_vertices, Nat.0).contains(x)
} by {
    if six_vertices.contains(x) and x != Nat.0 {
        neighborhood_contains_eq(complete_graph[Nat], six_vertices, Nat.0, x)
        (neighborhood(complete_graph[Nat], six_vertices, Nat.0).contains(x)
            = (six_vertices.contains(x) and complete_graph[Nat].adj(Nat.0, x)))
        complete_graph_adj_iff_ne[Nat](Nat.0, x)
        complete_graph[Nat].adj(Nat.0, x) = (Nat.0 != x)
        six_vertices.contains(x)
        complete_graph[Nat].adj(Nat.0, x)
        neighborhood(complete_graph[Nat], six_vertices, Nat.0).contains(x)
    }
}

/// A 2-coloring of the edges of a graph: every edge is either red (`true`) or blue (`false`).
///
/// Edges are unordered, so a coloring must agree on the two directions of an edge.
define is_edge_2_coloring[V](g: SimpleGraph[V], color: (V, V) -> Bool) -> Bool {
    forall(x: V, y: V) {
        g.adj(x, y) implies color(x, y) = color(y, x)
    }
}

/// The two directions of an edge carry the same color.
theorem edge_2_coloring_comm[V](g: SimpleGraph[V], color: (V, V) -> Bool, x: V, y: V) {
    is_edge_2_coloring(g, color) and g.adj(x, y) implies color(x, y) = color(y, x)
} by {
    if is_edge_2_coloring(g, color) and g.adj(x, y) {
        is_edge_2_coloring(g, color) = forall(a: V, b: V) {
            g.adj(a, b) implies color(a, b) = color(b, a)
        }
        forall(a: V, b: V) {
            g.adj(a, b) implies color(a, b) = color(b, a)
        }
        g.adj(x, y) implies color(x, y) = color(y, x)
        color(x, y) = color(y, x)
    }
}

/// True of the three distinct vertices of a red triangle in `s`.
define has_red_triangle[V](color: (V, V) -> Bool, s: FiniteSet[V]) -> Bool {
    exists(x: V, y: V, z: V) {
        s.contains(x) and s.contains(y) and s.contains(z) and x != y and x != z and y != z
        and color(x, y) and color(x, z) and color(y, z)
    }
}

/// Three vertices of `s` pairwise joined by red edges form a red triangle.
theorem has_red_triangle_intro[V](color: (V, V) -> Bool, s: FiniteSet[V], x: V, y: V, z: V) {
    s.contains(x) and s.contains(y) and s.contains(z) and x != y and x != z and y != z
    and color(x, y) and color(x, z) and color(y, z)
    implies has_red_triangle(color, s)
} by {
    if s.contains(x) and s.contains(y) and s.contains(z) and x != y and x != z and y != z
        and color(x, y) and color(x, z) and color(y, z) {
        has_red_triangle(color, s) = exists(a: V, b: V, c: V) {
            s.contains(a) and s.contains(b) and s.contains(c) and a != b and a != c and b != c
            and color(a, b) and color(a, c) and color(b, c)
        }
        exists_intro(function(a: V) {
            exists(b: V, c: V) {
                s.contains(a) and s.contains(b) and s.contains(c) and a != b and a != c and b != c
                and color(a, b) and color(a, c) and color(b, c)
            }
        }, x)
        exists_intro(function(b: V) {
            exists(c: V) {
                s.contains(x) and s.contains(b) and s.contains(c) and x != b and x != c and b != c
                and color(x, b) and color(x, c) and color(b, c)
            }
        }, y)
        exists_intro(function(c: V) {
            s.contains(x) and s.contains(y) and s.contains(c) and x != y and x != c and y != c
            and color(x, y) and color(x, c) and color(y, c)
        }, z)
        exists(a: V, b: V, c: V) {
            s.contains(a) and s.contains(b) and s.contains(c) and a != b and a != c and b != c
            and color(a, b) and color(a, c) and color(b, c)
        }
        has_red_triangle(color, s)
    }
}

/// True of the three distinct vertices of a blue triangle in `s`.
define has_blue_triangle[V](color: (V, V) -> Bool, s: FiniteSet[V]) -> Bool {
    exists(x: V, y: V, z: V) {
        s.contains(x) and s.contains(y) and s.contains(z) and x != y and x != z and y != z
        and not color(x, y) and not color(x, z) and not color(y, z)
    }
}

/// Three vertices of `s` pairwise joined by blue edges form a blue triangle.
theorem has_blue_triangle_intro[V](color: (V, V) -> Bool, s: FiniteSet[V], x: V, y: V, z: V) {
    s.contains(x) and s.contains(y) and s.contains(z) and x != y and x != z and y != z
    and not color(x, y) and not color(x, z) and not color(y, z)
    implies has_blue_triangle(color, s)
} by {
    if s.contains(x) and s.contains(y) and s.contains(z) and x != y and x != z and y != z
        and not color(x, y) and not color(x, z) and not color(y, z) {
        has_blue_triangle(color, s) = exists(a: V, b: V, c: V) {
            s.contains(a) and s.contains(b) and s.contains(c) and a != b and a != c and b != c
            and not color(a, b) and not color(a, c) and not color(b, c)
        }
        exists_intro(function(a: V) {
            exists(b: V, c: V) {
                s.contains(a) and s.contains(b) and s.contains(c) and a != b and a != c and b != c
                and not color(a, b) and not color(a, c) and not color(b, c)
            }
        }, x)
        exists_intro(function(b: V) {
            exists(c: V) {
                s.contains(x) and s.contains(b) and s.contains(c) and x != b and x != c and b != c
                and not color(x, b) and not color(x, c) and not color(b, c)
            }
        }, y)
        exists_intro(function(c: V) {
            s.contains(x) and s.contains(y) and s.contains(c) and x != y and x != c and y != c
            and not color(x, y) and not color(x, c) and not color(y, c)
        }, z)
        exists(a: V, b: V, c: V) {
            s.contains(a) and s.contains(b) and s.contains(c) and a != b and a != c and b != c
            and not color(a, b) and not color(a, c) and not color(b, c)
        }
        has_blue_triangle(color, s)
    }
}

/// True if some triangle of `s` is monochromatic: all red or all blue.
define has_mono_triangle[V](color: (V, V) -> Bool, s: FiniteSet[V]) -> Bool {
    has_red_triangle(color, s) or has_blue_triangle(color, s)
}

/// True if three of the five edges from vertex 0 to 1, 2, 3, 4, 5 share a color.
///
/// The vertices are recorded as distinct members of the six-element set other than 0, which is
/// everything the triangle argument below needs to know about them.
define three_edges_from_zero_share_color(color: (Nat, Nat) -> Bool) -> Bool {
    exists(x: Nat, y: Nat, z: Nat) {
        six_vertices.contains(x) and six_vertices.contains(y) and six_vertices.contains(z)
        and x != Nat.0 and y != Nat.0 and z != Nat.0
        and x != y and x != z and y != z
        and color(Nat.0, x) = color(Nat.0, y) and color(Nat.0, y) = color(Nat.0, z)
    }
}

/// Three vertices with equal edge colors to 0 witness the pigeonhole.
theorem three_edges_from_zero_share_color_intro(color: (Nat, Nat) -> Bool, x: Nat, y: Nat, z: Nat) {
    six_vertices.contains(x) and six_vertices.contains(y) and six_vertices.contains(z)
    and x != Nat.0 and y != Nat.0 and z != Nat.0
    and x != y and x != z and y != z
    and color(Nat.0, x) = color(Nat.0, y) and color(Nat.0, y) = color(Nat.0, z)
    implies three_edges_from_zero_share_color(color)
} by {
    if six_vertices.contains(x) and six_vertices.contains(y) and six_vertices.contains(z)
        and x != Nat.0 and y != Nat.0 and z != Nat.0
        and x != y and x != z and y != z
        and color(Nat.0, x) = color(Nat.0, y) and color(Nat.0, y) = color(Nat.0, z) {
        three_edges_from_zero_share_color(color) = exists(a: Nat, b: Nat, c: Nat) {
            six_vertices.contains(a) and six_vertices.contains(b) and six_vertices.contains(c)
            and a != Nat.0 and b != Nat.0 and c != Nat.0
            and a != b and a != c and b != c
            and color(Nat.0, a) = color(Nat.0, b) and color(Nat.0, b) = color(Nat.0, c)
        }
        exists_intro(function(a: Nat) {
            exists(b: Nat, c: Nat) {
                six_vertices.contains(a) and six_vertices.contains(b) and six_vertices.contains(c)
                and a != Nat.0 and b != Nat.0 and c != Nat.0
                and a != b and a != c and b != c
                and color(Nat.0, a) = color(Nat.0, b) and color(Nat.0, b) = color(Nat.0, c)
            }
        }, x)
        exists_intro(function(b: Nat) {
            exists(c: Nat) {
                six_vertices.contains(x) and six_vertices.contains(b) and six_vertices.contains(c)
                and x != Nat.0 and b != Nat.0 and c != Nat.0
                and x != b and x != c and b != c
                and color(Nat.0, x) = color(Nat.0, b) and color(Nat.0, b) = color(Nat.0, c)
            }
        }, y)
        exists_intro(function(c: Nat) {
            six_vertices.contains(x) and six_vertices.contains(y) and six_vertices.contains(c)
            and x != Nat.0 and y != Nat.0 and c != Nat.0
            and x != y and x != c and y != c
            and color(Nat.0, x) = color(Nat.0, y) and color(Nat.0, y) = color(Nat.0, c)
        }, z)
        exists(a: Nat, b: Nat, c: Nat) {
            six_vertices.contains(a) and six_vertices.contains(b) and six_vertices.contains(c)
            and a != Nat.0 and b != Nat.0 and c != Nat.0
            and a != b and a != c and b != c
            and color(Nat.0, a) = color(Nat.0, b) and color(Nat.0, b) = color(Nat.0, c)
        }
        three_edges_from_zero_share_color(color)
    }
}

/// The pigeonhole conclusion can be unpacked.
theorem three_edges_from_zero_share_color_witness(color: (Nat, Nat) -> Bool) {
    three_edges_from_zero_share_color(color) implies exists(x: Nat, y: Nat, z: Nat) {
        six_vertices.contains(x) and six_vertices.contains(y) and six_vertices.contains(z)
        and x != Nat.0 and y != Nat.0 and z != Nat.0
        and x != y and x != z and y != z
        and color(Nat.0, x) = color(Nat.0, y) and color(Nat.0, y) = color(Nat.0, z)
    }
} by {
    if three_edges_from_zero_share_color(color) {
        three_edges_from_zero_share_color(color) = exists(a: Nat, b: Nat, c: Nat) {
            six_vertices.contains(a) and six_vertices.contains(b) and six_vertices.contains(c)
            and a != Nat.0 and b != Nat.0 and c != Nat.0
            and a != b and a != c and b != c
            and color(Nat.0, a) = color(Nat.0, b) and color(Nat.0, b) = color(Nat.0, c)
        }
        exists(a: Nat, b: Nat, c: Nat) {
            six_vertices.contains(a) and six_vertices.contains(b) and six_vertices.contains(c)
            and a != Nat.0 and b != Nat.0 and c != Nat.0
            and a != b and a != c and b != c
            and color(Nat.0, a) = color(Nat.0, b) and color(Nat.0, b) = color(Nat.0, c)
        }
    }
}

// (the pigeonhole case analysis and the main theorem follow below)

/// Among the five edges from vertex 0, three share a color.
///
/// The five edge colors are case-split directly; each branch names three edges of a common
/// color. Only two colors exist, so five edges must contain three of one color.
theorem three_edges_from_zero_share_color_pigeonhole(color: (Nat, Nat) -> Bool) {
    three_edges_from_zero_share_color(color)
} by {
    six_labels_lt
    (Nat.0 < Nat.1 and Nat.1 < Nat.2 and Nat.2 < Nat.3 and Nat.3 < Nat.4 and Nat.4 < Nat.5
        and Nat.5 < Nat.6 and Nat.1 < Nat.6 and Nat.2 < Nat.6 and Nat.3 < Nat.6 and Nat.4 < Nat.6)
    six_vertices_contains(Nat.1)
    six_vertices_contains(Nat.2)
    six_vertices_contains(Nat.3)
    six_vertices_contains(Nat.4)
    six_vertices_contains(Nat.5)
    six_vertices.contains(Nat.1)
    six_vertices.contains(Nat.2)
    six_vertices.contains(Nat.3)
    six_vertices.contains(Nat.4)
    six_vertices.contains(Nat.5)
    six_labels_distinct
    if color(Nat.0, Nat.1) {
        if color(Nat.0, Nat.2) {
            if color(Nat.0, Nat.3) {
                if color(Nat.0, Nat.4) {
                    if color(Nat.0, Nat.5) {
                        (Nat.0 != Nat.1 and Nat.0 != Nat.2 and Nat.0 != Nat.3 and Nat.1 != Nat.2 and Nat.1 != Nat.3 and Nat.2 != Nat.3)
                        color(Nat.0, Nat.1) = color(Nat.0, Nat.2)
                        color(Nat.0, Nat.2) = color(Nat.0, Nat.3)
                        three_edges_from_zero_share_color_intro(color, Nat.1, Nat.2, Nat.3)
                        three_edges_from_zero_share_color(color)
                    } else {
                        (Nat.0 != Nat.1 and Nat.0 != Nat.2 and Nat.0 != Nat.3 and Nat.1 != Nat.2 and Nat.1 != Nat.3 and Nat.2 != Nat.3)
                        color(Nat.0, Nat.1) = color(Nat.0, Nat.2)
                        color(Nat.0, Nat.2) = color(Nat.0, Nat.3)
                        three_edges_from_zero_share_color_intro(color, Nat.1, Nat.2, Nat.3)
                        three_edges_from_zero_share_color(color)
                    }
                } else {
                    if color(Nat.0, Nat.5) {
                        (Nat.0 != Nat.1 and Nat.0 != Nat.2 and Nat.0 != Nat.3 and Nat.1 != Nat.2 and Nat.1 != Nat.3 and Nat.2 != Nat.3)
                        color(Nat.0, Nat.1) = color(Nat.0, Nat.2)
                        color(Nat.0, Nat.2) = color(Nat.0, Nat.3)
                        three_edges_from_zero_share_color_intro(color, Nat.1, Nat.2, Nat.3)
                        three_edges_from_zero_share_color(color)
                    } else {
                        (Nat.0 != Nat.1 and Nat.0 != Nat.2 and Nat.0 != Nat.3 and Nat.1 != Nat.2 and Nat.1 != Nat.3 and Nat.2 != Nat.3)
                        color(Nat.0, Nat.1) = color(Nat.0, Nat.2)
                        color(Nat.0, Nat.2) = color(Nat.0, Nat.3)
                        three_edges_from_zero_share_color_intro(color, Nat.1, Nat.2, Nat.3)
                        three_edges_from_zero_share_color(color)
                    }
                }
            } else {
                if color(Nat.0, Nat.4) {
                    if color(Nat.0, Nat.5) {
                        (Nat.0 != Nat.1 and Nat.0 != Nat.2 and Nat.0 != Nat.4 and Nat.1 != Nat.2 and Nat.1 != Nat.4 and Nat.2 != Nat.4)
                        color(Nat.0, Nat.1) = color(Nat.0, Nat.2)
                        color(Nat.0, Nat.2) = color(Nat.0, Nat.4)
                        three_edges_from_zero_share_color_intro(color, Nat.1, Nat.2, Nat.4)
                        three_edges_from_zero_share_color(color)
                    } else {
                        (Nat.0 != Nat.1 and Nat.0 != Nat.2 and Nat.0 != Nat.4 and Nat.1 != Nat.2 and Nat.1 != Nat.4 and Nat.2 != Nat.4)
                        color(Nat.0, Nat.1) = color(Nat.0, Nat.2)
                        color(Nat.0, Nat.2) = color(Nat.0, Nat.4)
                        three_edges_from_zero_share_color_intro(color, Nat.1, Nat.2, Nat.4)
                        three_edges_from_zero_share_color(color)
                    }
                } else {
                    if color(Nat.0, Nat.5) {
                        (Nat.0 != Nat.1 and Nat.0 != Nat.2 and Nat.0 != Nat.5 and Nat.1 != Nat.2 and Nat.1 != Nat.5 and Nat.2 != Nat.5)
                        color(Nat.0, Nat.1) = color(Nat.0, Nat.2)
                        color(Nat.0, Nat.2) = color(Nat.0, Nat.5)
                        three_edges_from_zero_share_color_intro(color, Nat.1, Nat.2, Nat.5)
                        three_edges_from_zero_share_color(color)
                    } else {
                        (Nat.0 != Nat.3 and Nat.0 != Nat.4 and Nat.0 != Nat.5 and Nat.3 != Nat.4 and Nat.3 != Nat.5 and Nat.4 != Nat.5)
                        color(Nat.0, Nat.3) = color(Nat.0, Nat.4)
                        color(Nat.0, Nat.4) = color(Nat.0, Nat.5)
                        three_edges_from_zero_share_color_intro(color, Nat.3, Nat.4, Nat.5)
                        three_edges_from_zero_share_color(color)
                    }
                }
            }
        } else {
            if color(Nat.0, Nat.3) {
                if color(Nat.0, Nat.4) {
                    if color(Nat.0, Nat.5) {
                        (Nat.0 != Nat.1 and Nat.0 != Nat.3 and Nat.0 != Nat.4 and Nat.1 != Nat.3 and Nat.1 != Nat.4 and Nat.3 != Nat.4)
                        color(Nat.0, Nat.1) = color(Nat.0, Nat.3)
                        color(Nat.0, Nat.3) = color(Nat.0, Nat.4)
                        three_edges_from_zero_share_color_intro(color, Nat.1, Nat.3, Nat.4)
                        three_edges_from_zero_share_color(color)
                    } else {
                        (Nat.0 != Nat.1 and Nat.0 != Nat.3 and Nat.0 != Nat.4 and Nat.1 != Nat.3 and Nat.1 != Nat.4 and Nat.3 != Nat.4)
                        color(Nat.0, Nat.1) = color(Nat.0, Nat.3)
                        color(Nat.0, Nat.3) = color(Nat.0, Nat.4)
                        three_edges_from_zero_share_color_intro(color, Nat.1, Nat.3, Nat.4)
                        three_edges_from_zero_share_color(color)
                    }
                } else {
                    if color(Nat.0, Nat.5) {
                        (Nat.0 != Nat.1 and Nat.0 != Nat.3 and Nat.0 != Nat.5 and Nat.1 != Nat.3 and Nat.1 != Nat.5 and Nat.3 != Nat.5)
                        color(Nat.0, Nat.1) = color(Nat.0, Nat.3)
                        color(Nat.0, Nat.3) = color(Nat.0, Nat.5)
                        three_edges_from_zero_share_color_intro(color, Nat.1, Nat.3, Nat.5)
                        three_edges_from_zero_share_color(color)
                    } else {
                        (Nat.0 != Nat.2 and Nat.0 != Nat.4 and Nat.0 != Nat.5 and Nat.2 != Nat.4 and Nat.2 != Nat.5 and Nat.4 != Nat.5)
                        color(Nat.0, Nat.2) = color(Nat.0, Nat.4)
                        color(Nat.0, Nat.4) = color(Nat.0, Nat.5)
                        three_edges_from_zero_share_color_intro(color, Nat.2, Nat.4, Nat.5)
                        three_edges_from_zero_share_color(color)
                    }
                }
            } else {
                if color(Nat.0, Nat.4) {
                    if color(Nat.0, Nat.5) {
                        (Nat.0 != Nat.1 and Nat.0 != Nat.4 and Nat.0 != Nat.5 and Nat.1 != Nat.4 and Nat.1 != Nat.5 and Nat.4 != Nat.5)
                        color(Nat.0, Nat.1) = color(Nat.0, Nat.4)
                        color(Nat.0, Nat.4) = color(Nat.0, Nat.5)
                        three_edges_from_zero_share_color_intro(color, Nat.1, Nat.4, Nat.5)
                        three_edges_from_zero_share_color(color)
                    } else {
                        (Nat.0 != Nat.2 and Nat.0 != Nat.3 and Nat.0 != Nat.5 and Nat.2 != Nat.3 and Nat.2 != Nat.5 and Nat.3 != Nat.5)
                        color(Nat.0, Nat.2) = color(Nat.0, Nat.3)
                        color(Nat.0, Nat.3) = color(Nat.0, Nat.5)
                        three_edges_from_zero_share_color_intro(color, Nat.2, Nat.3, Nat.5)
                        three_edges_from_zero_share_color(color)
                    }
                } else {
                    if color(Nat.0, Nat.5) {
                        (Nat.0 != Nat.2 and Nat.0 != Nat.3 and Nat.0 != Nat.4 and Nat.2 != Nat.3 and Nat.2 != Nat.4 and Nat.3 != Nat.4)
                        color(Nat.0, Nat.2) = color(Nat.0, Nat.3)
                        color(Nat.0, Nat.3) = color(Nat.0, Nat.4)
                        three_edges_from_zero_share_color_intro(color, Nat.2, Nat.3, Nat.4)
                        three_edges_from_zero_share_color(color)
                    } else {
                        (Nat.0 != Nat.2 and Nat.0 != Nat.3 and Nat.0 != Nat.4 and Nat.2 != Nat.3 and Nat.2 != Nat.4 and Nat.3 != Nat.4)
                        color(Nat.0, Nat.2) = color(Nat.0, Nat.3)
                        color(Nat.0, Nat.3) = color(Nat.0, Nat.4)
                        three_edges_from_zero_share_color_intro(color, Nat.2, Nat.3, Nat.4)
                        three_edges_from_zero_share_color(color)
                    }
                }
            }
        }
    } else {
        if color(Nat.0, Nat.2) {
            if color(Nat.0, Nat.3) {
                if color(Nat.0, Nat.4) {
                    if color(Nat.0, Nat.5) {
                        (Nat.0 != Nat.2 and Nat.0 != Nat.3 and Nat.0 != Nat.4 and Nat.2 != Nat.3 and Nat.2 != Nat.4 and Nat.3 != Nat.4)
                        color(Nat.0, Nat.2) = color(Nat.0, Nat.3)
                        color(Nat.0, Nat.3) = color(Nat.0, Nat.4)
                        three_edges_from_zero_share_color_intro(color, Nat.2, Nat.3, Nat.4)
                        three_edges_from_zero_share_color(color)
                    } else {
                        (Nat.0 != Nat.2 and Nat.0 != Nat.3 and Nat.0 != Nat.4 and Nat.2 != Nat.3 and Nat.2 != Nat.4 and Nat.3 != Nat.4)
                        color(Nat.0, Nat.2) = color(Nat.0, Nat.3)
                        color(Nat.0, Nat.3) = color(Nat.0, Nat.4)
                        three_edges_from_zero_share_color_intro(color, Nat.2, Nat.3, Nat.4)
                        three_edges_from_zero_share_color(color)
                    }
                } else {
                    if color(Nat.0, Nat.5) {
                        (Nat.0 != Nat.2 and Nat.0 != Nat.3 and Nat.0 != Nat.5 and Nat.2 != Nat.3 and Nat.2 != Nat.5 and Nat.3 != Nat.5)
                        color(Nat.0, Nat.2) = color(Nat.0, Nat.3)
                        color(Nat.0, Nat.3) = color(Nat.0, Nat.5)
                        three_edges_from_zero_share_color_intro(color, Nat.2, Nat.3, Nat.5)
                        three_edges_from_zero_share_color(color)
                    } else {
                        (Nat.0 != Nat.1 and Nat.0 != Nat.4 and Nat.0 != Nat.5 and Nat.1 != Nat.4 and Nat.1 != Nat.5 and Nat.4 != Nat.5)
                        color(Nat.0, Nat.1) = color(Nat.0, Nat.4)
                        color(Nat.0, Nat.4) = color(Nat.0, Nat.5)
                        three_edges_from_zero_share_color_intro(color, Nat.1, Nat.4, Nat.5)
                        three_edges_from_zero_share_color(color)
                    }
                }
            } else {
                if color(Nat.0, Nat.4) {
                    if color(Nat.0, Nat.5) {
                        (Nat.0 != Nat.2 and Nat.0 != Nat.4 and Nat.0 != Nat.5 and Nat.2 != Nat.4 and Nat.2 != Nat.5 and Nat.4 != Nat.5)
                        color(Nat.0, Nat.2) = color(Nat.0, Nat.4)
                        color(Nat.0, Nat.4) = color(Nat.0, Nat.5)
                        three_edges_from_zero_share_color_intro(color, Nat.2, Nat.4, Nat.5)
                        three_edges_from_zero_share_color(color)
                    } else {
                        (Nat.0 != Nat.1 and Nat.0 != Nat.3 and Nat.0 != Nat.5 and Nat.1 != Nat.3 and Nat.1 != Nat.5 and Nat.3 != Nat.5)
                        color(Nat.0, Nat.1) = color(Nat.0, Nat.3)
                        color(Nat.0, Nat.3) = color(Nat.0, Nat.5)
                        three_edges_from_zero_share_color_intro(color, Nat.1, Nat.3, Nat.5)
                        three_edges_from_zero_share_color(color)
                    }
                } else {
                    if color(Nat.0, Nat.5) {
                        (Nat.0 != Nat.1 and Nat.0 != Nat.3 and Nat.0 != Nat.4 and Nat.1 != Nat.3 and Nat.1 != Nat.4 and Nat.3 != Nat.4)
                        color(Nat.0, Nat.1) = color(Nat.0, Nat.3)
                        color(Nat.0, Nat.3) = color(Nat.0, Nat.4)
                        three_edges_from_zero_share_color_intro(color, Nat.1, Nat.3, Nat.4)
                        three_edges_from_zero_share_color(color)
                    } else {
                        (Nat.0 != Nat.1 and Nat.0 != Nat.3 and Nat.0 != Nat.4 and Nat.1 != Nat.3 and Nat.1 != Nat.4 and Nat.3 != Nat.4)
                        color(Nat.0, Nat.1) = color(Nat.0, Nat.3)
                        color(Nat.0, Nat.3) = color(Nat.0, Nat.4)
                        three_edges_from_zero_share_color_intro(color, Nat.1, Nat.3, Nat.4)
                        three_edges_from_zero_share_color(color)
                    }
                }
            }
        } else {
            if color(Nat.0, Nat.3) {
                if color(Nat.0, Nat.4) {
                    if color(Nat.0, Nat.5) {
                        (Nat.0 != Nat.3 and Nat.0 != Nat.4 and Nat.0 != Nat.5 and Nat.3 != Nat.4 and Nat.3 != Nat.5 and Nat.4 != Nat.5)
                        color(Nat.0, Nat.3) = color(Nat.0, Nat.4)
                        color(Nat.0, Nat.4) = color(Nat.0, Nat.5)
                        three_edges_from_zero_share_color_intro(color, Nat.3, Nat.4, Nat.5)
                        three_edges_from_zero_share_color(color)
                    } else {
                        (Nat.0 != Nat.1 and Nat.0 != Nat.2 and Nat.0 != Nat.5 and Nat.1 != Nat.2 and Nat.1 != Nat.5 and Nat.2 != Nat.5)
                        color(Nat.0, Nat.1) = color(Nat.0, Nat.2)
                        color(Nat.0, Nat.2) = color(Nat.0, Nat.5)
                        three_edges_from_zero_share_color_intro(color, Nat.1, Nat.2, Nat.5)
                        three_edges_from_zero_share_color(color)
                    }
                } else {
                    if color(Nat.0, Nat.5) {
                        (Nat.0 != Nat.1 and Nat.0 != Nat.2 and Nat.0 != Nat.4 and Nat.1 != Nat.2 and Nat.1 != Nat.4 and Nat.2 != Nat.4)
                        color(Nat.0, Nat.1) = color(Nat.0, Nat.2)
                        color(Nat.0, Nat.2) = color(Nat.0, Nat.4)
                        three_edges_from_zero_share_color_intro(color, Nat.1, Nat.2, Nat.4)
                        three_edges_from_zero_share_color(color)
                    } else {
                        (Nat.0 != Nat.1 and Nat.0 != Nat.2 and Nat.0 != Nat.4 and Nat.1 != Nat.2 and Nat.1 != Nat.4 and Nat.2 != Nat.4)
                        color(Nat.0, Nat.1) = color(Nat.0, Nat.2)
                        color(Nat.0, Nat.2) = color(Nat.0, Nat.4)
                        three_edges_from_zero_share_color_intro(color, Nat.1, Nat.2, Nat.4)
                        three_edges_from_zero_share_color(color)
                    }
                }
            } else {
                if color(Nat.0, Nat.4) {
                    if color(Nat.0, Nat.5) {
                        (Nat.0 != Nat.1 and Nat.0 != Nat.2 and Nat.0 != Nat.3 and Nat.1 != Nat.2 and Nat.1 != Nat.3 and Nat.2 != Nat.3)
                        color(Nat.0, Nat.1) = color(Nat.0, Nat.2)
                        color(Nat.0, Nat.2) = color(Nat.0, Nat.3)
                        three_edges_from_zero_share_color_intro(color, Nat.1, Nat.2, Nat.3)
                        three_edges_from_zero_share_color(color)
                    } else {
                        (Nat.0 != Nat.1 and Nat.0 != Nat.2 and Nat.0 != Nat.3 and Nat.1 != Nat.2 and Nat.1 != Nat.3 and Nat.2 != Nat.3)
                        color(Nat.0, Nat.1) = color(Nat.0, Nat.2)
                        color(Nat.0, Nat.2) = color(Nat.0, Nat.3)
                        three_edges_from_zero_share_color_intro(color, Nat.1, Nat.2, Nat.3)
                        three_edges_from_zero_share_color(color)
                    }
                } else {
                    if color(Nat.0, Nat.5) {
                        (Nat.0 != Nat.1 and Nat.0 != Nat.2 and Nat.0 != Nat.3 and Nat.1 != Nat.2 and Nat.1 != Nat.3 and Nat.2 != Nat.3)
                        color(Nat.0, Nat.1) = color(Nat.0, Nat.2)
                        color(Nat.0, Nat.2) = color(Nat.0, Nat.3)
                        three_edges_from_zero_share_color_intro(color, Nat.1, Nat.2, Nat.3)
                        three_edges_from_zero_share_color(color)
                    } else {
                        (Nat.0 != Nat.1 and Nat.0 != Nat.2 and Nat.0 != Nat.3 and Nat.1 != Nat.2 and Nat.1 != Nat.3 and Nat.2 != Nat.3)
                        color(Nat.0, Nat.1) = color(Nat.0, Nat.2)
                        color(Nat.0, Nat.2) = color(Nat.0, Nat.3)
                        three_edges_from_zero_share_color_intro(color, Nat.1, Nat.2, Nat.3)
                        three_edges_from_zero_share_color(color)
                    }
                }
            }
        }
    }
}

/// Ramsey's theorem R(3, 3) <= 6: every red/blue 2-coloring of the edges of the
/// complete graph on six vertices contains a monochromatic triangle.
///
/// Fix vertex 0. Among its five incident edges three share a color (pigeonhole); let
/// `p < q < r`-ordered witnesses be `p`, `q`, `r`. If two of the edges among `p`, `q`,
/// `r` share the color of the edges to 0, that pair closes a triangle with 0; otherwise
/// the three edges among `p`, `q`, `r` are all of the other color, which is a triangle
/// on its own.
theorem ramsey_r33_upper(color: (Nat, Nat) -> Bool) {
    is_edge_2_coloring(complete_graph[Nat], color) implies
    has_mono_triangle(color, six_vertices)
} by {
    if is_edge_2_coloring(complete_graph[Nat], color) {
        three_edges_from_zero_share_color_pigeonhole(color)
        three_edges_from_zero_share_color(color)
        three_edges_from_zero_share_color_witness(color)
        let (p: Nat, q: Nat, r: Nat) satisfy {
            six_vertices.contains(p) and six_vertices.contains(q) and six_vertices.contains(r)
            and p != Nat.0 and q != Nat.0 and r != Nat.0
            and p != q and p != r and q != r
            and color(Nat.0, p) = color(Nat.0, q) and color(Nat.0, q) = color(Nat.0, r)
        }
        six_labels_lt
        (Nat.0 < Nat.1 and Nat.1 < Nat.2 and Nat.2 < Nat.3 and Nat.3 < Nat.4 and Nat.4 < Nat.5
            and Nat.5 < Nat.6 and Nat.0 < Nat.6 and Nat.1 < Nat.6 and Nat.2 < Nat.6
            and Nat.3 < Nat.6 and Nat.4 < Nat.6)
        six_vertices_contains(Nat.0)
        six_vertices.contains(Nat.0)
        has_mono_triangle(color, six_vertices) =
            (has_red_triangle(color, six_vertices) or has_blue_triangle(color, six_vertices))
        if color(Nat.0, p) {
            color(Nat.0, p)
            color(Nat.0, q)
            color(Nat.0, r)
            if color(p, q) {
                has_red_triangle_intro(color, six_vertices, Nat.0, p, q)
                has_red_triangle(color, six_vertices)
                has_mono_triangle(color, six_vertices)
            } else {
                if color(p, r) {
                    has_red_triangle_intro(color, six_vertices, Nat.0, p, r)
                    has_red_triangle(color, six_vertices)
                    has_mono_triangle(color, six_vertices)
                } else {
                    if color(q, r) {
                        has_red_triangle_intro(color, six_vertices, Nat.0, q, r)
                        has_red_triangle(color, six_vertices)
                        has_mono_triangle(color, six_vertices)
                    } else {
                        has_blue_triangle_intro(color, six_vertices, p, q, r)
                        has_blue_triangle(color, six_vertices)
                        has_mono_triangle(color, six_vertices)
                    }
                }
            }
        } else {
            if color(Nat.0, q) {
                color(Nat.0, q) = color(Nat.0, p)
                color(Nat.0, p)
                false
            }
            not color(Nat.0, q)
            if color(Nat.0, r) {
                color(Nat.0, q) = color(Nat.0, r)
                color(Nat.0, q)
                false
            }
            not color(Nat.0, r)
            if not color(p, q) {
                has_blue_triangle_intro(color, six_vertices, Nat.0, p, q)
                has_blue_triangle(color, six_vertices)
                has_mono_triangle(color, six_vertices)
            } else {
                if not color(p, r) {
                    has_blue_triangle_intro(color, six_vertices, Nat.0, p, r)
                    has_blue_triangle(color, six_vertices)
                    has_mono_triangle(color, six_vertices)
                } else {
                    if not color(q, r) {
                        has_blue_triangle_intro(color, six_vertices, Nat.0, q, r)
                        has_blue_triangle(color, six_vertices)
                        has_mono_triangle(color, six_vertices)
                    } else {
                        has_red_triangle_intro(color, six_vertices, p, q, r)
                        has_red_triangle(color, six_vertices)
                        has_mono_triangle(color, six_vertices)
                    }
                }
            }
        }
    }
}


// ============================================================================
