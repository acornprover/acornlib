from nat import Nat, lt_and_lte, lte_and_lt, lte_trans, lte_antisymm
from finite_set import FiniteSet, finite_set_ext
from data.finite.finite_set_card import fs_card
from data.nat.nat_range_set import range_set, range_set_contains, range_set_contains_eq, range_set_lt,
    range_set_subset, range_set_card
from graph.simple_graph import SimpleGraph
from graph.simple_graph_path import path_graph, path_graph_adj_iff, path_graph_adj_suc,
    path_graph_neighbor_cases
from graph.simple_graph_zero_forcing_rule import is_white_neighbor, can_force, can_force_apply,
    can_force_intro, is_forceable, is_forceable_intro, is_forceable_witness, derived_set,
    derived_set_contains_eq
from graph.simple_graph_zero_forcing_closure import forcing_iterate, forcing_iterate_zero,
    forcing_iterate_suc, is_zero_forcing_set, is_zero_forcing_set_intro
from graph.simple_graph_zero_forcing_number import zero_forcing_number, zero_forcing_number_is_least
from graph.simple_graph_zero_forcing_lower import zero_forcing_number_pos

numerals Nat

/// The last blue vertex of a prefix forces the next one.
///
/// Its neighbours are the vertex before it, which is already blue, and the vertex after it,
/// which is white; so the white one is unique and the rule applies.
theorem path_prefix_can_force(n: Nat, k: Nat) {
    k.suc < n implies can_force(path_graph, range_set(n), range_set(k.suc), k, k.suc)
} by {
    if k.suc < n {
        k < k.suc
        range_set_contains(k.suc, k)
        range_set(k.suc).contains(k)
        range_set_contains(n, k.suc)
        range_set(n).contains(k.suc)
        range_set_contains_eq(k.suc, k.suc)
        range_set(k.suc).contains(k.suc) = (k.suc < k.suc)
        not range_set(k.suc).contains(k.suc)
        path_graph_adj_suc(k)
        path_graph.adj(k, k.suc)
        (is_white_neighbor(path_graph, range_set(n), range_set(k.suc), k, k.suc)
            = (range_set(n).contains(k.suc) and not range_set(k.suc).contains(k.suc)
                and path_graph.adj(k, k.suc)))
        is_white_neighbor(path_graph, range_set(n), range_set(k.suc), k, k.suc)
        forall(w: Nat) {
            if is_white_neighbor(path_graph, range_set(n), range_set(k.suc), k, w) {
                (is_white_neighbor(path_graph, range_set(n), range_set(k.suc), k, w)
                    = (range_set(n).contains(w) and not range_set(k.suc).contains(w)
                        and path_graph.adj(k, w)))
                not range_set(k.suc).contains(w)
                range_set_contains_eq(k.suc, w)
                range_set(k.suc).contains(w) = (w < k.suc)
                not (w < k.suc)
                path_graph.adj(k, w)
                path_graph_neighbor_cases(k, w)
                (w = k.suc or w.suc = k)
                if w.suc = k {
                    w < w.suc
                    w < k
                    k < k.suc
                    lt_and_lte(w, k, k.suc)
                    w < k.suc
                    false
                }
                w = k.suc
            }
            (is_white_neighbor(path_graph, range_set(n), range_set(k.suc), k, w)
                implies w = k.suc)
        }
        can_force_intro(path_graph, range_set(n), range_set(k.suc), k, k.suc)
        can_force(path_graph, range_set(n), range_set(k.suc), k, k.suc)
    }
}

/// One round of forcing extends a blue prefix by exactly one vertex.
///
/// Nothing further along can be forced: a blue vertex is at most `k`, and its only neighbours
/// are `k - 1` and `k + 1`, so no vertex beyond `k + 1` is anyone's white neighbour.
theorem path_derived_set_step(n: Nat, k: Nat) {
    k.suc < n implies
        (derived_set(path_graph, range_set(n), range_set(k.suc)) = range_set(k.suc.suc))
} by {
    if k.suc < n {
        forall(v: Nat) {
            if derived_set(path_graph, range_set(n), range_set(k.suc)).contains(v) {
                derived_set_contains_eq(path_graph, range_set(n), range_set(k.suc), v)
                (range_set(n).contains(v)
                    and (range_set(k.suc).contains(v)
                        or is_forceable(path_graph, range_set(n), range_set(k.suc), v)))
                if range_set(k.suc).contains(v) {
                    range_set_lt(k.suc, v)
                    v < k.suc
                    k.suc < k.suc.suc
                    lt_and_lte(v, k.suc, k.suc.suc)
                    v < k.suc.suc
                }
                if not range_set(k.suc).contains(v) {
                    is_forceable(path_graph, range_set(n), range_set(k.suc), v)
                    is_forceable_witness(path_graph, range_set(n), range_set(k.suc), v)
                    let (u: Nat) satisfy {
                        can_force(path_graph, range_set(n), range_set(k.suc), u, v)
                    }
                    can_force_apply(path_graph, range_set(n), range_set(k.suc), u, v)
                    range_set(k.suc).contains(u)
                    is_white_neighbor(path_graph, range_set(n), range_set(k.suc), u, v)
                    (is_white_neighbor(path_graph, range_set(n), range_set(k.suc), u, v)
                        = (range_set(n).contains(v) and not range_set(k.suc).contains(v)
                            and path_graph.adj(u, v)))
                    path_graph.adj(u, v)
                    range_set_lt(k.suc, u)
                    u < k.suc
                    path_graph_neighbor_cases(u, v)
                    (v = u.suc or v.suc = u)
                    if v.suc = u {
                        v < v.suc
                        v < u
                        lt_and_lte(v, u, k.suc)
                        v < k.suc
                        range_set_contains(k.suc, v)
                        range_set(k.suc).contains(v)
                        false
                    }
                    v = u.suc
                    u.suc <= k.suc
                    v <= k.suc
                    k.suc < k.suc.suc
                    lte_and_lt(v, k.suc, k.suc.suc)
                    v < k.suc.suc
                }
                v < k.suc.suc
                range_set_contains(k.suc.suc, v)
                range_set(k.suc.suc).contains(v)
            }
            if range_set(k.suc.suc).contains(v) {
                range_set_lt(k.suc.suc, v)
                v < k.suc.suc
                v <= k.suc
                (v < k.suc or v = k.suc)
                k.suc < n
                if v < k.suc {
                    range_set_contains(k.suc, v)
                    range_set(k.suc).contains(v)
                    lt_and_lte(v, k.suc, n)
                    v < n
                    range_set_contains(n, v)
                    range_set(n).contains(v)
                    derived_set_contains_eq(path_graph, range_set(n), range_set(k.suc), v)
                    derived_set(path_graph, range_set(n), range_set(k.suc)).contains(v)
                }
                if v = k.suc {
                    path_prefix_can_force(n, k)
                    can_force(path_graph, range_set(n), range_set(k.suc), k, k.suc)
                    is_forceable_intro(path_graph, range_set(n), range_set(k.suc), k, k.suc)
                    is_forceable(path_graph, range_set(n), range_set(k.suc), k.suc)
                    is_forceable(path_graph, range_set(n), range_set(k.suc), v)
                    range_set_contains(n, k.suc)
                    range_set(n).contains(v)
                    derived_set_contains_eq(path_graph, range_set(n), range_set(k.suc), v)
                    derived_set(path_graph, range_set(n), range_set(k.suc)).contains(v)
                }
                derived_set(path_graph, range_set(n), range_set(k.suc)).contains(v)
            }
            (derived_set(path_graph, range_set(n), range_set(k.suc)).contains(v)
                implies range_set(k.suc.suc).contains(v))
            (range_set(k.suc.suc).contains(v)
                implies derived_set(path_graph, range_set(n), range_set(k.suc)).contains(v))
            (derived_set(path_graph, range_set(n), range_set(k.suc)).contains(v)
                = range_set(k.suc.suc).contains(v))
        }
        finite_set_ext(derived_set(path_graph, range_set(n), range_set(k.suc)),
            range_set(k.suc.suc))
        (derived_set(path_graph, range_set(n), range_set(k.suc)) = range_set(k.suc.suc))
    }
}

/// After `k` rounds the blue set is the prefix of length `k + 1`.
theorem path_forcing_iterate_prefix(n: Nat, k: Nat) {
    k.suc <= n implies
        (forcing_iterate(path_graph, range_set(n), range_set(Nat.1), k) = range_set(k.suc))
} by {
    define p(x: Nat) -> Bool {
        x.suc <= n implies
            (forcing_iterate(path_graph, range_set(n), range_set(Nat.1), x)
                = range_set(x.suc))
    }
    forcing_iterate_zero(path_graph, range_set(n), range_set(Nat.1))
    forcing_iterate(path_graph, range_set(n), range_set(Nat.1), Nat.0) = range_set(Nat.1)
    Nat.0.suc = Nat.1
    p(Nat.0)
    forall(m: Nat) {
        if p(m) {
            if m.suc.suc <= n {
                m.suc <= m.suc.suc
                lte_trans(m.suc, m.suc.suc, n)
                m.suc <= n
                (m.suc <= n implies
                    (forcing_iterate(path_graph, range_set(n), range_set(Nat.1), m)
                        = range_set(m.suc)))
                (forcing_iterate(path_graph, range_set(n), range_set(Nat.1), m)
                    = range_set(m.suc))
                forcing_iterate_suc(path_graph, range_set(n), range_set(Nat.1), m)
                (forcing_iterate(path_graph, range_set(n), range_set(Nat.1), m.suc)
                    = derived_set(path_graph, range_set(n),
                        forcing_iterate(path_graph, range_set(n), range_set(Nat.1), m)))
                (forcing_iterate(path_graph, range_set(n), range_set(Nat.1), m.suc)
                    = derived_set(path_graph, range_set(n), range_set(m.suc)))
                m.suc < m.suc.suc
                lt_and_lte(m.suc, m.suc.suc, n)
                m.suc < n
                path_derived_set_step(n, m)
                derived_set(path_graph, range_set(n), range_set(m.suc)) = range_set(m.suc.suc)
                (forcing_iterate(path_graph, range_set(n), range_set(Nat.1), m.suc)
                    = range_set(m.suc.suc))
            }
            p(m.suc)
        }
        (p(m) implies p(m.suc))
    }
    p(Nat.0) and forall(m: Nat) {
        p(m) implies p(m.suc)
    }
    Nat.induction(p)
    p(k)
}

/// A single endpoint is a zero forcing set of any path.
///
/// Colouring vertex `0` blue and applying the rule `n - 1` times colours the whole path: at
/// each stage the last blue vertex has exactly one white neighbour, the next one along.
theorem path_singleton_is_zero_forcing(n: Nat) {
    Nat.1 <= n implies is_zero_forcing_set(path_graph, range_set(n.suc), range_set(Nat.1))
} by {
    if Nat.1 <= n {
        n <= n.suc
        lte_trans(Nat.1, n, n.suc)
        Nat.1 <= n.suc
        range_set_subset(Nat.1, n.suc)
        range_set(Nat.1).subset_eq(range_set(n.suc))
        n.suc <= n.suc
        path_forcing_iterate_prefix(n.suc, n)
        forcing_iterate(path_graph, range_set(n.suc), range_set(Nat.1), n) = range_set(n.suc)
        is_zero_forcing_set_intro(path_graph, range_set(n.suc), range_set(Nat.1), n)
        is_zero_forcing_set(path_graph, range_set(n.suc), range_set(Nat.1))
    }
}

/// The zero forcing number of a path is at most one.
///
/// The classical value is exactly one for a nonempty path; this is the upper bound, which is
/// the direction that needs the forcing argument.
theorem path_zero_forcing_number_le_one(n: Nat) {
    Nat.1 <= n implies zero_forcing_number(path_graph, range_set(n.suc)) <= Nat.1
} by {
    if Nat.1 <= n {
        path_singleton_is_zero_forcing(n)
        is_zero_forcing_set(path_graph, range_set(n.suc), range_set(Nat.1))
        zero_forcing_number_is_least(path_graph, range_set(n.suc), range_set(Nat.1))
        zero_forcing_number(path_graph, range_set(n.suc)) <= fs_card(range_set(Nat.1))
        range_set_card(Nat.1)
        fs_card(range_set(Nat.1)) = Nat.1
        zero_forcing_number(path_graph, range_set(n.suc)) <= Nat.1
    }
}

/// The zero forcing number of a nonempty path is exactly one.
///
/// The upper bound is the forcing argument above; the lower bound is that some vertex has to
/// start blue. `P_n` is the path on the vertex set below `n`, so this is stated at `n + 1` to
/// keep it nonempty.
theorem path_zero_forcing_number(n: Nat) {
    Nat.1 <= n implies zero_forcing_number(path_graph, range_set(n.suc)) = Nat.1
} by {
    if Nat.1 <= n {
        path_zero_forcing_number_le_one(n)
        zero_forcing_number(path_graph, range_set(n.suc)) <= Nat.1
        Nat.0 < n.suc
        range_set_contains(n.suc, Nat.0)
        range_set(n.suc).contains(Nat.0)
        zero_forcing_number_pos(path_graph, range_set(n.suc), Nat.0)
        Nat.1 <= zero_forcing_number(path_graph, range_set(n.suc))
        lte_antisymm(zero_forcing_number(path_graph, range_set(n.suc)), Nat.1)
        zero_forcing_number(path_graph, range_set(n.suc)) = Nat.1
    }
}
