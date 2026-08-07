from nat import Nat
from finite_set import FiniteSet, finite_set_ext, finite_set_empty_subset,
    finite_set_empty_contains_eq
from data.finite.finite_set_card import fs_card
from data.finite.finite_set_card_members import fs_card_pos_of_contains
from graph.simple_graph import SimpleGraph
from graph.simple_graph_zero_forcing_rule import can_force, can_force_apply, can_force_unique,
    is_white_neighbor, is_forceable, is_forceable_witness, is_forcing_closed,
    is_forcing_closed_intro
from graph.simple_graph_zero_forcing_closed import forcing_iterate_constant_of_closed
from graph.simple_graph_zero_forcing_closure import forcing_iterate, is_zero_forcing_set,
    is_zero_forcing_set_apply
from graph.simple_graph_zero_forcing_number import zero_forcing_number, zero_forcing_number_attained,
    zero_forcing_size_pred

numerals Nat

/// A colouring with no blue vertex forces nothing.
///
/// The rule needs a blue vertex to do the forcing, so with none the colouring is already
/// closed.
theorem no_blue_is_forcing_closed[V](g: SimpleGraph[V], s: FiniteSet[V], b: FiniteSet[V]) {
    (forall(x: V) { not b.contains(x) }) implies is_forcing_closed(g, s, b)
} by {
    if forall(x: V) { not b.contains(x) } {
        forall(v: V) {
            if is_forceable(g, s, b, v) {
                is_forceable_witness(g, s, b, v)
                let (u: V) satisfy {
                    can_force(g, s, b, u, v)
                }
                can_force_apply(g, s, b, u, v)
                b.contains(u)
                not b.contains(u)
                false
            }
            not is_forceable(g, s, b, v)
        }
        is_forcing_closed_intro(g, s, b)
        is_forcing_closed(g, s, b)
    }
}

/// A zero forcing set of a nonempty vertex set has a member.
///
/// Starting from no blue vertices the colouring never changes, so it never reaches a vertex
/// set that has something in it.
theorem zero_forcing_set_nonempty[V](
    g: SimpleGraph[V], s: FiniteSet[V], b: FiniteSet[V], x: V
) {
    s.contains(x) and is_zero_forcing_set(g, s, b)
        implies exists(y: V) { b.contains(y) }
} by {
    if s.contains(x) and is_zero_forcing_set(g, s, b) {
        if not exists(y: V) { b.contains(y) } {
            forall(z: V) {
                not b.contains(z)
            }
            no_blue_is_forcing_closed(g, s, b)
            is_forcing_closed(g, s, b)
            forall(z: V) {
                not b.contains(z)
                finite_set_empty_contains_eq(z)
                not FiniteSet.empty[V].contains(z)
                b.contains(z) = FiniteSet.empty[V].contains(z)
            }
            finite_set_ext(b, FiniteSet.empty[V])
            b = FiniteSet.empty[V]
            finite_set_empty_subset(s)
            FiniteSet.empty[V].subset_eq(s)
            b.subset_eq(s)
            is_zero_forcing_set_apply(g, s, b)
            exists(n: Nat) { forcing_iterate(g, s, b, n) = s }
            let (n: Nat) satisfy {
                forcing_iterate(g, s, b, n) = s
            }
            forcing_iterate_constant_of_closed(g, s, b, n)
            forcing_iterate(g, s, b, n) = b
            b = s
            b.contains(x)
            not b.contains(x)
            false
        }
        exists(y: V) { b.contains(y) }
    }
}

/// The zero forcing number of a nonempty vertex set is at least one.
///
/// The lower bound that pairs with any construction: some vertex has to start blue.
theorem zero_forcing_number_pos[V](g: SimpleGraph[V], s: FiniteSet[V], x: V) {
    s.contains(x) implies Nat.1 <= zero_forcing_number(g, s)
} by {
    if s.contains(x) {
        zero_forcing_number_attained(g, s)
        zero_forcing_size_pred(g, s)(zero_forcing_number(g, s))
        (zero_forcing_size_pred(g, s)(zero_forcing_number(g, s))
            = exists(b: FiniteSet[V]) {
                is_zero_forcing_set(g, s, b) and fs_card(b) = zero_forcing_number(g, s)
            })
        let (b: FiniteSet[V]) satisfy {
            is_zero_forcing_set(g, s, b) and fs_card(b) = zero_forcing_number(g, s)
        }
        zero_forcing_set_nonempty(g, s, b, x)
        exists(y: V) { b.contains(y) }
        let (y: V) satisfy {
            b.contains(y)
        }
        fs_card_pos_of_contains(b, y)
        Nat.1 <= fs_card(b)
        Nat.1 <= zero_forcing_number(g, s)
    }
}

/// True if `u` has two distinct white neighbours.
///
/// Named rather than written inline so the blocking lemma below has a two-variable hypothesis;
/// stated with the quantifier inside, the statement is too deep for proof search to assemble.
define has_two_white_neighbors[V](
    g: SimpleGraph[V], s: FiniteSet[V], b: FiniteSet[V], u: V
) -> Bool {
    exists(w1: V, w2: V) {
        w1 != w2 and is_white_neighbor(g, s, b, u, w1) and is_white_neighbor(g, s, b, u, w2)
    }
}

/// Two distinct white neighbours give the predicate.
theorem has_two_white_neighbors_intro[V](
    g: SimpleGraph[V], s: FiniteSet[V], b: FiniteSet[V], u: V, w1: V, w2: V
) {
    w1 != w2 and is_white_neighbor(g, s, b, u, w1) and is_white_neighbor(g, s, b, u, w2)
        implies has_two_white_neighbors(g, s, b, u)
} by {
    if w1 != w2 and is_white_neighbor(g, s, b, u, w1) and is_white_neighbor(g, s, b, u, w2) {
        (has_two_white_neighbors(g, s, b, u) = exists(x: V, y: V) {
            x != y and is_white_neighbor(g, s, b, u, x) and is_white_neighbor(g, s, b, u, y)
        })
        exists(x: V, y: V) {
            x != y and is_white_neighbor(g, s, b, u, x) and is_white_neighbor(g, s, b, u, y)
        }
        has_two_white_neighbors(g, s, b, u)
    }
}

/// The predicate produces the two neighbours.
theorem has_two_white_neighbors_apply[V](
    g: SimpleGraph[V], s: FiniteSet[V], b: FiniteSet[V], u: V
) {
    has_two_white_neighbors(g, s, b, u) implies exists(w1: V, w2: V) {
        w1 != w2 and is_white_neighbor(g, s, b, u, w1) and is_white_neighbor(g, s, b, u, w2)
    }
} by {
    if has_two_white_neighbors(g, s, b, u) {
        (has_two_white_neighbors(g, s, b, u) = exists(x: V, y: V) {
            x != y and is_white_neighbor(g, s, b, u, x) and is_white_neighbor(g, s, b, u, y)
        })
        exists(x: V, y: V) {
            x != y and is_white_neighbor(g, s, b, u, x) and is_white_neighbor(g, s, b, u, y)
        }
    }
}

/// A colouring where every blue vertex has two white neighbours forces nothing.
///
/// The rule's uniqueness clause fails at every blue vertex, so nothing is forceable and the
/// colouring is closed. This is the general form of the obstruction that stops a single seed
/// on a cycle and two white vertices on a complete graph.
theorem two_white_neighbors_blocks_forcing[V](
    g: SimpleGraph[V], s: FiniteSet[V], b: FiniteSet[V]
) {
    (forall(u: V) { b.contains(u) implies has_two_white_neighbors(g, s, b, u) })
        implies is_forcing_closed(g, s, b)
} by {
    if forall(u: V) { b.contains(u) implies has_two_white_neighbors(g, s, b, u) } {
        forall(v: V) {
            if is_forceable(g, s, b, v) {
                is_forceable_witness(g, s, b, v)
                let (u: V) satisfy {
                    can_force(g, s, b, u, v)
                }
                can_force_apply(g, s, b, u, v)
                b.contains(u)
                (b.contains(u) implies has_two_white_neighbors(g, s, b, u))
                has_two_white_neighbors(g, s, b, u)
                has_two_white_neighbors_apply(g, s, b, u)
                exists(x: V, y: V) {
                    x != y and is_white_neighbor(g, s, b, u, x)
                        and is_white_neighbor(g, s, b, u, y)
                }
                let (w1: V, w2: V) satisfy {
                    w1 != w2 and is_white_neighbor(g, s, b, u, w1)
                        and is_white_neighbor(g, s, b, u, w2)
                }
                can_force_unique(g, s, b, u, v, w1)
                w1 = v
                can_force_unique(g, s, b, u, v, w2)
                w2 = v
                w1 = w2
                false
            }
            not is_forceable(g, s, b, v)
        }
        is_forcing_closed_intro(g, s, b)
        is_forcing_closed(g, s, b)
    }
}
