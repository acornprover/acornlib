from nat import Nat, lte_trans, lt_and_lte
from pair import Pair
from finite_set import FiniteSet
from data.finite.finite_set_card import fs_card
from data.finite.finite_set_product import finite_set_product, finite_set_product_contains_pair
from data.finite.finite_set_product_card import fs_card_product
from data.finite.finite_set_cover_card import is_covered_by_image, is_covered_by_image_intro,
    fs_card_le_of_covered_by_image, has_preimage_in, has_preimage_in_intro
from data.nat.nat_range_set import range_set, range_set_contains, range_set_card
from graph.simple_graph_domination import is_dominated, is_dominating_set, is_dominating_set_apply,
    has_neighbor_in, has_neighbor_in_witness
from graph.simple_graph_domination_number import domination_number, domination_number_attained,
    dominating_size_pred
from graph.simple_graph_path import path_graph, path_graph_adj_iff

numerals Nat

/// The vertex reached from a class and an offset.
///
/// A vertex of the path dominated by `w` is one of `w - 1`, `w`, and `w + 1`, which are
/// `w + 0 - 1`, `w + 1 - 1`, and `w + 2 - 1`. Writing them this way keeps every offset a
/// natural, so the three cases are one expression indexed by `{0, 1, 2}`, and the truncated
/// subtraction is exact in each case that arises.
define offset_vertex(p: Pair[Nat, Nat]) -> Nat {
    p.first + p.second - Nat.1
}

/// The three offsets lie below three.
///
/// Comparisons between numerals are not free, so the ladder is stated once.
theorem three_offsets_lt {
    Nat.0 < Nat.3 and Nat.1 < Nat.3 and Nat.2 < Nat.3
} by {
    Nat.2.suc = Nat.3
    Nat.2 < Nat.2.suc
    Nat.2 < Nat.3
    Nat.1.suc = Nat.2
    Nat.1 < Nat.1.suc
    Nat.1 < Nat.2
    Nat.2 <= Nat.3
    lt_and_lte(Nat.1, Nat.2, Nat.3)
    Nat.1 < Nat.3
    Nat.0.suc = Nat.1
    Nat.0 < Nat.0.suc
    Nat.0 < Nat.1
    Nat.1 <= Nat.3
    lt_and_lte(Nat.0, Nat.1, Nat.3)
    Nat.0 < Nat.3
}

/// A dominated vertex of the path is reached from its dominator by one of three offsets.
theorem path_dominated_is_offset(d: FiniteSet[Nat], v: Nat) {
    is_dominated(path_graph, d, v)
        implies has_preimage_in(finite_set_product(d, range_set(Nat.3)), offset_vertex, v)
} by {
    if is_dominated(path_graph, d, v) {
        (is_dominated(path_graph, d, v)
            = (d.contains(v) or has_neighbor_in(path_graph, d, v)))
        if d.contains(v) {
            three_offsets_lt
            (Nat.1 < Nat.3)
            range_set_contains(Nat.3, Nat.1)
            range_set(Nat.3).contains(Nat.1)
            finite_set_product_contains_pair(d, range_set(Nat.3), v, Nat.1)
            finite_set_product(d, range_set(Nat.3)).contains(Pair.new(v, Nat.1))
            (Pair.new(v, Nat.1).first = v)
            (Pair.new(v, Nat.1).second = Nat.1)
            (offset_vertex(Pair.new(v, Nat.1)) = v + Nat.1 - Nat.1)
            (v + Nat.1 - Nat.1 = v)
            offset_vertex(Pair.new(v, Nat.1)) = v
            has_preimage_in_intro(finite_set_product(d, range_set(Nat.3)), offset_vertex, v,
                Pair.new(v, Nat.1))
            has_preimage_in(finite_set_product(d, range_set(Nat.3)), offset_vertex, v)
        }
        if has_neighbor_in(path_graph, d, v) {
            has_neighbor_in_witness(path_graph, d, v)
            exists(w: Nat) { d.contains(w) and path_graph.adj(w, v) }
            let (w: Nat) satisfy {
                d.contains(w) and path_graph.adj(w, v)
            }
            path_graph_adj_iff(w, v)
            (path_graph.adj(w, v) = (w.suc = v or v.suc = w))
            if w.suc = v {
                three_offsets_lt
                (Nat.2 < Nat.3)
                range_set_contains(Nat.3, Nat.2)
                range_set(Nat.3).contains(Nat.2)
                finite_set_product_contains_pair(d, range_set(Nat.3), w, Nat.2)
                finite_set_product(d, range_set(Nat.3)).contains(Pair.new(w, Nat.2))
                (Pair.new(w, Nat.2).first = w)
                (Pair.new(w, Nat.2).second = Nat.2)
                (offset_vertex(Pair.new(w, Nat.2)) = w + Nat.2 - Nat.1)
                (w + Nat.2 = w.suc + Nat.1)
                (w.suc + Nat.1 - Nat.1 = w.suc)
                offset_vertex(Pair.new(w, Nat.2)) = v
                has_preimage_in_intro(finite_set_product(d, range_set(Nat.3)), offset_vertex,
                    v, Pair.new(w, Nat.2))
                has_preimage_in(finite_set_product(d, range_set(Nat.3)), offset_vertex, v)
            }
            if v.suc = w {
                three_offsets_lt
                (Nat.0 < Nat.3)
                range_set_contains(Nat.3, Nat.0)
                range_set(Nat.3).contains(Nat.0)
                finite_set_product_contains_pair(d, range_set(Nat.3), w, Nat.0)
                finite_set_product(d, range_set(Nat.3)).contains(Pair.new(w, Nat.0))
                (Pair.new(w, Nat.0).first = w)
                (Pair.new(w, Nat.0).second = Nat.0)
                (offset_vertex(Pair.new(w, Nat.0)) = w + Nat.0 - Nat.1)
                (w + Nat.0 = w)
                (w = v + Nat.1)
                (v + Nat.1 - Nat.1 = v)
                offset_vertex(Pair.new(w, Nat.0)) = v
                has_preimage_in_intro(finite_set_product(d, range_set(Nat.3)), offset_vertex,
                    v, Pair.new(w, Nat.0))
                has_preimage_in(finite_set_product(d, range_set(Nat.3)), offset_vertex, v)
            }
            has_preimage_in(finite_set_product(d, range_set(Nat.3)), offset_vertex, v)
        }
        has_preimage_in(finite_set_product(d, range_set(Nat.3)), offset_vertex, v)
    }
}

/// A dominating set of a path covers it three vertices at a time.
theorem path_range_covered_by_dominating(n: Nat, d: FiniteSet[Nat]) {
    is_dominating_set(path_graph, range_set(n), d)
        implies is_covered_by_image(range_set(n),
            finite_set_product(d, range_set(Nat.3)), offset_vertex)
} by {
    if is_dominating_set(path_graph, range_set(n), d) {
        forall(v: Nat) {
            if range_set(n).contains(v) {
                is_dominating_set_apply(path_graph, range_set(n), d, v)
                is_dominated(path_graph, d, v)
                path_dominated_is_offset(d, v)
                has_preimage_in(finite_set_product(d, range_set(Nat.3)), offset_vertex, v)
            }
            (range_set(n).contains(v) implies
                has_preimage_in(finite_set_product(d, range_set(Nat.3)), offset_vertex, v))
        }
        is_covered_by_image_intro(range_set(n), finite_set_product(d, range_set(Nat.3)),
            offset_vertex)
        is_covered_by_image(range_set(n), finite_set_product(d, range_set(Nat.3)),
            offset_vertex)
    }
}

/// A dominating set of a path has at least a third as many vertices as the path.
///
/// Each vertex of the dominating set accounts for at most three vertices of the path, itself
/// and its two neighbors, and together they account for all of them.
theorem path_dominating_set_card_bound(n: Nat, d: FiniteSet[Nat]) {
    is_dominating_set(path_graph, range_set(n), d) implies n <= Nat.3 * fs_card(d)
} by {
    if is_dominating_set(path_graph, range_set(n), d) {
        path_range_covered_by_dominating(n, d)
        is_covered_by_image(range_set(n), finite_set_product(d, range_set(Nat.3)),
            offset_vertex)
        fs_card_le_of_covered_by_image(range_set(n),
            finite_set_product(d, range_set(Nat.3)), offset_vertex)
        fs_card(range_set(n)) <= fs_card(finite_set_product(d, range_set(Nat.3)))
        fs_card_product(d, range_set(Nat.3))
        (fs_card(finite_set_product(d, range_set(Nat.3)))
            = fs_card(range_set(Nat.3)) * fs_card(d))
        range_set_card(Nat.3)
        fs_card(range_set(Nat.3)) = Nat.3
        range_set_card(n)
        fs_card(range_set(n)) = n
        n <= Nat.3 * fs_card(d)
    }
}

/// The domination number of a path is at least a third of its length.
///
/// The classical lower bound. The path has maximum degree two, so a vertex dominates at most
/// three vertices, and a dominating set must account for all of them.
theorem path_domination_number_lower_bound(n: Nat) {
    n <= Nat.3 * domination_number(path_graph, range_set(n))
} by {
    domination_number_attained(path_graph, range_set(n))
    dominating_size_pred(path_graph, range_set(n))(
        domination_number(path_graph, range_set(n)))
    (dominating_size_pred(path_graph, range_set(n))(
        domination_number(path_graph, range_set(n))) = exists(d: FiniteSet[Nat]) {
        d.subset_eq(range_set(n)) and is_dominating_set(path_graph, range_set(n), d)
            and fs_card(d) = domination_number(path_graph, range_set(n))
    })
    exists(d: FiniteSet[Nat]) {
        d.subset_eq(range_set(n)) and is_dominating_set(path_graph, range_set(n), d)
            and fs_card(d) = domination_number(path_graph, range_set(n))
    }
    let (e: FiniteSet[Nat]) satisfy {
        e.subset_eq(range_set(n)) and is_dominating_set(path_graph, range_set(n), e)
            and fs_card(e) = domination_number(path_graph, range_set(n))
    }
    path_dominating_set_card_bound(n, e)
    n <= Nat.3 * fs_card(e)
    n <= Nat.3 * domination_number(path_graph, range_set(n))
}
