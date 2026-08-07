from nat import Nat
from pair import Pair
from finite_set import FiniteSet
from data.finite.finite_set_product import finite_set_product, finite_set_product_contains_eq,
    finite_set_product_contains_pair
from data.finite.finite_set_filter import finite_set_filter, finite_set_filter_contains_eq,
    finite_set_filter_subset
from data.finite.finite_set_card import fs_card
from graph.simple_graph import SimpleGraph, simple_graph_adj_comm, simple_graph_adj_ne,
    simple_graph_adj_irreflexive

numerals Nat

/// True of the pairs whose components are adjacent.
///
/// Packaged as a predicate on pairs so it can cut a subset out of a product.
define adjacent_pair[V](g: SimpleGraph[V]) -> (Pair[V, V] -> Bool) {
    function(p: Pair[V, V]) {
        g.adj(p.first, p.second)
    }
}

/// A pair is adjacent exactly when its components are.
theorem adjacent_pair_eq[V](g: SimpleGraph[V], p: Pair[V, V]) {
    adjacent_pair(g)(p) = g.adj(p.first, p.second)
}

/// The ordered pairs of adjacent vertices drawn from `s`.
///
/// Each undirected edge appears twice here, once in each orientation. Counting
/// ordered pairs avoids choosing a representative for an unordered pair, which a
/// vertex type with no ordering cannot supply.
define directed_edges[V](g: SimpleGraph[V], s: FiniteSet[V]) -> FiniteSet[Pair[V, V]] {
    finite_set_filter(finite_set_product(s, s), adjacent_pair(g))
}

/// A pair is a directed edge exactly when both components lie in `s` and are adjacent.
theorem directed_edges_contains_eq[V](
    g: SimpleGraph[V], s: FiniteSet[V], p: Pair[V, V]
) {
    directed_edges(g, s).contains(p) = (s.contains(p.first) and s.contains(p.second) and g.adj(p.first, p.second))
} by {
    finite_set_filter_contains_eq(finite_set_product(s, s), adjacent_pair(g), p)
    directed_edges(g, s).contains(p) = (finite_set_product(s, s).contains(p) and adjacent_pair(g)(p))
    finite_set_product_contains_eq(s, s, p)
    finite_set_product(s, s).contains(p) = (s.contains(p.first) and s.contains(p.second))
    adjacent_pair_eq(g, p)
    adjacent_pair(g)(p) = g.adj(p.first, p.second)
}

/// Adjacent vertices of `s` give a directed edge.
theorem directed_edges_contains_pair[V](
    g: SimpleGraph[V], s: FiniteSet[V], x: V, y: V
) {
    s.contains(x) and s.contains(y) and g.adj(x, y) implies directed_edges(g, s).contains(Pair.new(x, y))
} by {
    if s.contains(x) and s.contains(y) and g.adj(x, y) {
        directed_edges_contains_eq(g, s, Pair.new(x, y))
        Pair.new(x, y).first = x
        Pair.new(x, y).second = y
        directed_edges(g, s).contains(Pair.new(x, y))
    }
}

/// Directed edges sit inside the product of the vertex set with itself.
theorem directed_edges_subset[V](g: SimpleGraph[V], s: FiniteSet[V]) {
    directed_edges(g, s).subset_eq(finite_set_product(s, s))
} by {
    finite_set_filter_subset(finite_set_product(s, s), adjacent_pair(g))
}

/// The first component of a directed edge lies in the vertex set.
theorem directed_edges_first[V](g: SimpleGraph[V], s: FiniteSet[V], p: Pair[V, V]) {
    directed_edges(g, s).contains(p) implies s.contains(p.first)
} by {
    directed_edges_contains_eq(g, s, p)
}

/// The second component of a directed edge lies in the vertex set.
theorem directed_edges_second[V](g: SimpleGraph[V], s: FiniteSet[V], p: Pair[V, V]) {
    directed_edges(g, s).contains(p) implies s.contains(p.second)
} by {
    directed_edges_contains_eq(g, s, p)
}

/// The components of a directed edge are adjacent.
theorem directed_edges_adj[V](g: SimpleGraph[V], s: FiniteSet[V], p: Pair[V, V]) {
    directed_edges(g, s).contains(p) implies g.adj(p.first, p.second)
} by {
    directed_edges_contains_eq(g, s, p)
}

/// The components of a directed edge are distinct, since no vertex is self-adjacent.
theorem directed_edges_ne[V](g: SimpleGraph[V], s: FiniteSet[V], p: Pair[V, V]) {
    directed_edges(g, s).contains(p) implies p.first != p.second
} by {
    if directed_edges(g, s).contains(p) {
        directed_edges_adj(g, s, p)
        g.adj(p.first, p.second)
        simple_graph_adj_ne(g, p.first, p.second)
    }
}

/// Reversing a directed edge gives a directed edge.
///
/// This is the involution that makes the directed count twice the undirected one.
theorem directed_edges_reverse[V](
    g: SimpleGraph[V], s: FiniteSet[V], x: V, y: V
) {
    directed_edges(g, s).contains(Pair.new(x, y)) implies directed_edges(g, s).contains(Pair.new(y, x))
} by {
    if directed_edges(g, s).contains(Pair.new(x, y)) {
        directed_edges_contains_eq(g, s, Pair.new(x, y))
        Pair.new(x, y).first = x
        Pair.new(x, y).second = y
        s.contains(x)
        s.contains(y)
        g.adj(x, y)
        simple_graph_adj_comm(g, x, y)
        g.adj(y, x)
        directed_edges_contains_pair(g, s, y, x)
        directed_edges(g, s).contains(Pair.new(y, x))
    }
}

/// No pair of a vertex with itself is a directed edge.
theorem directed_edges_no_loop[V](g: SimpleGraph[V], s: FiniteSet[V], v: V) {
    not directed_edges(g, s).contains(Pair.new(v, v))
} by {
    if directed_edges(g, s).contains(Pair.new(v, v)) {
        directed_edges_ne(g, s, Pair.new(v, v))
        Pair.new(v, v).first = v
        Pair.new(v, v).second = v
        v != v
        false
    }
    not directed_edges(g, s).contains(Pair.new(v, v))
}

/// The number of ordered adjacent pairs drawn from `s`.
///
/// Twice the number of edges, since each edge is counted in both orientations.
define directed_edge_count[V](g: SimpleGraph[V], s: FiniteSet[V]) -> Nat {
    fs_card(directed_edges(g, s))
}

/// The directed edge count is the size of the directed edge set.
theorem directed_edge_count_eq[V](g: SimpleGraph[V], s: FiniteSet[V]) {
    directed_edge_count(g, s) = fs_card(directed_edges(g, s))
}
