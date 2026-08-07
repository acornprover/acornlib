from nat import Nat, lte_antisymm, lte_cancel_suc
from finite_set import FiniteSet, finite_set_subset_contains, finite_set_subset_refl
from data.finite.finite_set_card import fs_card
from data.finite.finite_set_card_compare import fs_card_mono
from data.finite.finite_set_card_members import fs_two_distinct_members
from graph.simple_graph import SimpleGraph, complete_graph, complete_graph_adj_iff_ne
from graph.simple_graph_independent import is_clique_in, is_clique_in_intro, is_independent_in,
    is_independent_in_apply
from graph.simple_graph_independent_domination import is_independent_subset,
    is_independent_subset_apply
from graph.simple_graph_max_independent import independence_number, independence_number_attained,
    independent_size_pred, maximum_independent_set_exists
from graph.simple_graph_clique_number import is_clique_subset, is_clique_subset_apply,
    is_clique_subset_intro, clique_number, clique_number_attained, clique_number_is_greatest,
    clique_size_pred, clique_size_pred_witness

numerals Nat

/// The whole vertex set is a clique of the complete graph.
theorem complete_graph_whole_is_clique[V](s: FiniteSet[V]) {
    is_clique_subset(complete_graph[V], s, s)
} by {
    forall(x: V, y: V) {
        if s.contains(x) and s.contains(y) and x != y {
            complete_graph_adj_iff_ne(x, y)
            (complete_graph[V].adj(x, y) = (x != y))
            complete_graph[V].adj(x, y)
        }
        (s.contains(x) and s.contains(y) and x != y implies complete_graph[V].adj(x, y))
    }
    is_clique_in_intro(complete_graph[V], s)
    is_clique_in(complete_graph[V], s)
    finite_set_subset_refl(s)
    s.subset_eq(s)
    is_clique_subset_intro(complete_graph[V], s, s)
    is_clique_subset(complete_graph[V], s, s)
}

/// The clique number of the complete graph is the number of vertices.
///
/// The whole vertex set is a clique, and no clique subset is larger than the vertex set.
theorem complete_graph_clique_number[V](s: FiniteSet[V]) {
    clique_number(complete_graph[V], s) = fs_card(s)
} by {
    complete_graph_whole_is_clique(s)
    is_clique_subset(complete_graph[V], s, s)
    clique_number_is_greatest(complete_graph[V], s, s)
    fs_card(s) <= clique_number(complete_graph[V], s)
    clique_number_attained(complete_graph[V], s)
    clique_size_pred(complete_graph[V], s)(clique_number(complete_graph[V], s))
    clique_size_pred_witness(complete_graph[V], s, clique_number(complete_graph[V], s))
    exists(t: FiniteSet[V]) {
        is_clique_subset(complete_graph[V], s, t)
            and fs_card(t) = clique_number(complete_graph[V], s)
    }
    let (t: FiniteSet[V]) satisfy {
        is_clique_subset(complete_graph[V], s, t)
            and fs_card(t) = clique_number(complete_graph[V], s)
    }
    is_clique_subset_apply(complete_graph[V], s, t)
    (t.subset_eq(s) and is_clique_in(complete_graph[V], t))
    fs_card_mono(t, s)
    fs_card(t) <= fs_card(s)
    clique_number(complete_graph[V], s) <= fs_card(s)
    lte_antisymm(clique_number(complete_graph[V], s), fs_card(s))
    clique_number(complete_graph[V], s) = fs_card(s)
}

/// An independent subset of the complete graph has at most one vertex.
///
/// Two distinct vertices of the complete graph are adjacent, so no independent set holds two.
theorem complete_graph_independent_subset_card[V](s: FiniteSet[V], t: FiniteSet[V]) {
    is_independent_subset(complete_graph[V], s, t) implies fs_card(t) < Nat.2
} by {
    if is_independent_subset(complete_graph[V], s, t) {
        is_independent_subset_apply(complete_graph[V], s, t)
        (t.subset_eq(s) and is_independent_in(complete_graph[V], t))
        if Nat.2 <= fs_card(t) {
            fs_two_distinct_members(t)
            exists(a: V, b: V) {
                t.contains(a) and t.contains(b) and a != b
            }
            let (x: V, y: V) satisfy {
                t.contains(x) and t.contains(y) and x != y
            }
            is_independent_in_apply(complete_graph[V], t, x, y)
            not complete_graph[V].adj(x, y)
            complete_graph_adj_iff_ne(x, y)
            (complete_graph[V].adj(x, y) = (x != y))
            complete_graph[V].adj(x, y)
            false
        }
        fs_card(t) < Nat.2
    }
}

/// The independence number of the complete graph is at most one.
///
/// With the clique number above this is the complement duality made concrete: the complete
/// graph has everything as one clique and nothing but single vertices as independent sets, and
/// its complement, the empty graph, has these the other way round.
theorem complete_graph_independence_number_le_one[V](s: FiniteSet[V]) {
    independence_number(complete_graph[V], s) <= Nat.1
} by {
    maximum_independent_set_exists(complete_graph[V], s)
    exists(t: FiniteSet[V]) {
        is_independent_subset(complete_graph[V], s, t)
            and fs_card(t) = independence_number(complete_graph[V], s)
    }
    let (t: FiniteSet[V]) satisfy {
        is_independent_subset(complete_graph[V], s, t)
            and fs_card(t) = independence_number(complete_graph[V], s)
    }
    complete_graph_independent_subset_card(s, t)
    fs_card(t) < Nat.2
    independence_number(complete_graph[V], s) < Nat.2
    independence_number(complete_graph[V], s).suc <= Nat.2
    Nat.1.suc = Nat.2
    lte_cancel_suc(independence_number(complete_graph[V], s), Nat.1)
    independence_number(complete_graph[V], s) <= Nat.1
}
