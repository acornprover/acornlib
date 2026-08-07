from nat import Nat
from finite_set import FiniteSet, finite_set_subset_contains
from graph.simple_graph import SimpleGraph, simple_graph_adj_comm
from graph.simple_graph_degree import neighborhood, neighborhood_contains_eq
from graph.simple_graph_independent import is_independent_in, is_independent_in_intro
from graph.simple_graph_bipartite import is_bipartition, is_bipartition_apply
from graph.simple_graph_triangle_free import is_triangle_free, is_triangle_free_apply,
    is_triangle_free_intro, is_triangle_free_no_triangle, has_no_common_neighbor,
    has_no_common_neighbor_apply, has_no_common_neighbor_intro
from graph.simple_graph_vertex_sets import common_neighborhood, common_neighborhood_contains_eq

numerals Nat

/// Triangle-freeness passes to subsets of the vertex set.
///
/// The common neighborhood shrinks with the ambient set, so a pair without common
/// neighbors in the whole set has none in a subset either.
theorem is_triangle_free_of_subset[V](
    g: SimpleGraph[V], s: FiniteSet[V], t: FiniteSet[V]
) {
    t.subset_eq(s) and is_triangle_free(g, s) implies is_triangle_free(g, t)
} by {
    if t.subset_eq(s) and is_triangle_free(g, s) {
        forall(x: V, y: V) {
            if t.contains(x) and t.contains(y) and g.adj(x, y) {
                finite_set_subset_contains(t, s, x)
                finite_set_subset_contains(t, s, y)
                s.contains(x)
                s.contains(y)
                is_triangle_free_apply(g, s, x, y)
                has_no_common_neighbor(g, s, x, y)
                forall(z: V) {
                    if common_neighborhood(g, t, x, y).contains(z) {
                        common_neighborhood_contains_eq(g, t, x, y, z)
                        t.contains(z) and g.adj(x, z) and g.adj(y, z)
                        finite_set_subset_contains(t, s, z)
                        s.contains(z)
                        common_neighborhood_contains_eq(g, s, x, y, z)
                        common_neighborhood(g, s, x, y).contains(z)
                        has_no_common_neighbor_apply(g, s, x, y, z)
                        not common_neighborhood(g, s, x, y).contains(z)
                        false
                    }
                    not common_neighborhood(g, t, x, y).contains(z)
                }
                has_no_common_neighbor_intro(g, t, x, y)
                has_no_common_neighbor(g, t, x, y)
            }
            t.contains(x) and t.contains(y) and g.adj(x, y) implies has_no_common_neighbor(g, t, x, y)
        }
        is_triangle_free_intro(g, t)
        is_triangle_free(g, t)
    }
}

/// In a triangle-free graph, the neighborhood of a vertex is independent.
///
/// Two adjacent neighbors of `v` would close a triangle with `v`.
theorem triangle_free_neighborhood_independent[V](
    g: SimpleGraph[V], s: FiniteSet[V], v: V
) {
    is_triangle_free(g, s) and s.contains(v) implies is_independent_in(g, neighborhood(g, s, v))
} by {
    if is_triangle_free(g, s) and s.contains(v) {
        forall(x: V, y: V) {
            if neighborhood(g, s, v).contains(x) and neighborhood(g, s, v).contains(y) {
                neighborhood_contains_eq(g, s, v, x)
                s.contains(x) and g.adj(v, x)
                neighborhood_contains_eq(g, s, v, y)
                s.contains(y) and g.adj(v, y)
                is_triangle_free_no_triangle(g, s, v, x, y)
                not g.adj(x, y)
            }
            neighborhood(g, s, v).contains(x) and neighborhood(g, s, v).contains(y) implies not g.adj(x, y)
        }
        is_independent_in_intro(g, neighborhood(g, s, v))
        is_independent_in(g, neighborhood(g, s, v))
    }
}

/// A bipartite graph is triangle-free.
///
/// Three mutually adjacent vertices would need three pairwise different sides, but
/// a two-colouring has only two.
theorem bipartition_is_triangle_free[V](
    g: SimpleGraph[V], s: FiniteSet[V], part: V -> Bool
) {
    is_bipartition(g, s, part) implies is_triangle_free(g, s)
} by {
    if is_bipartition(g, s, part) {
        forall(x: V, y: V) {
            if s.contains(x) and s.contains(y) and g.adj(x, y) {
                is_bipartition_apply(g, s, part, x, y)
                part(x) != part(y)
                forall(z: V) {
                    if common_neighborhood(g, s, x, y).contains(z) {
                        common_neighborhood_contains_eq(g, s, x, y, z)
                        s.contains(z) and g.adj(x, z) and g.adj(y, z)
                        is_bipartition_apply(g, s, part, x, z)
                        part(x) != part(z)
                        is_bipartition_apply(g, s, part, y, z)
                        part(y) != part(z)
                        if part(x) {
                            not part(y)
                            not part(z)
                            part(y) = part(z)
                            false
                        }
                        if not part(x) {
                            part(y)
                            part(z)
                            part(y) = part(z)
                            false
                        }
                        false
                    }
                    not common_neighborhood(g, s, x, y).contains(z)
                }
                has_no_common_neighbor_intro(g, s, x, y)
                has_no_common_neighbor(g, s, x, y)
            }
            s.contains(x) and s.contains(y) and g.adj(x, y) implies has_no_common_neighbor(g, s, x, y)
        }
        is_triangle_free_intro(g, s)
        is_triangle_free(g, s)
    }
}
