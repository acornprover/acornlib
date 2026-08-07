from nat import Nat
from finite_set import FiniteSet, finite_set_subset_contains, finite_set_subset_refl,
    finite_set_subset_antisymm
from data.finite.finite_set_subset_intro import fs_subset_eq_intro
from data.finite.finite_set_membership import finite_set_eq_of_contains_eq
from graph.simple_graph import SimpleGraph
from graph.simple_graph_zero_forcing_rule import is_white_neighbor, can_force, can_force_apply,
    can_force_unique, can_force_intro, is_forceable, is_forceable_intro,
    is_forceable_witness, derived_set, derived_set_contains_eq
from graph.simple_graph_zero_forcing_closure import forcing_iterate, forcing_iterate_suc,
    forcing_iterate_subset, is_zero_forcing_set, is_zero_forcing_set_apply,
    is_zero_forcing_set_intro

numerals Nat

/// A white neighbor under a larger blue set is white under a smaller one.
///
/// Colouring more vertices blue can only remove white neighbors, never create them.
theorem is_white_neighbor_of_subset[V](
    g: SimpleGraph[V], s: FiniteSet[V], b: FiniteSet[V], c: FiniteSet[V], u: V, w: V
) {
    b.subset_eq(c) and is_white_neighbor(g, s, c, u, w)
        implies is_white_neighbor(g, s, b, u, w)
} by {
    if b.subset_eq(c) and is_white_neighbor(g, s, c, u, w) {
        is_white_neighbor(g, s, c, u, w) =
            (s.contains(w) and not c.contains(w) and g.adj(u, w))
        s.contains(w)
        not c.contains(w)
        g.adj(u, w)
        if b.contains(w) {
            finite_set_subset_contains(b, c, w)
            c.contains(w)
            false
        }
        not b.contains(w)
        is_white_neighbor(g, s, b, u, w) =
            (s.contains(w) and not b.contains(w) and g.adj(u, w))
        is_white_neighbor(g, s, b, u, w)
    }
}

/// A forcing pair survives colouring more vertices blue, as long as the forced vertex is
/// still white.
///
/// The forcing vertex stays blue, and its white neighbors under the larger set are among its
/// white neighbors under the smaller one, of which the forced vertex was the only one.
theorem can_force_of_subset[V](
    g: SimpleGraph[V], s: FiniteSet[V], b: FiniteSet[V], c: FiniteSet[V], u: V, v: V
) {
    b.subset_eq(c) and can_force(g, s, b, u, v) and not c.contains(v)
        implies can_force(g, s, c, u, v)
} by {
    if b.subset_eq(c) and can_force(g, s, b, u, v) and not c.contains(v) {
        can_force_apply(g, s, b, u, v)
        b.contains(u)
        finite_set_subset_contains(b, c, u)
        c.contains(u)
        is_white_neighbor(g, s, b, u, v)
        is_white_neighbor(g, s, b, u, v) =
            (s.contains(v) and not b.contains(v) and g.adj(u, v))
        s.contains(v)
        g.adj(u, v)
        is_white_neighbor(g, s, c, u, v) =
            (s.contains(v) and not c.contains(v) and g.adj(u, v))
        is_white_neighbor(g, s, c, u, v)
        forall(w: V) {
            if is_white_neighbor(g, s, c, u, w) {
                is_white_neighbor_of_subset(g, s, b, c, u, w)
                is_white_neighbor(g, s, b, u, w)
                can_force_unique(g, s, b, u, v, w)
                w = v
            }
            (is_white_neighbor(g, s, c, u, w) implies w = v)
        }
        can_force_intro(g, s, c, u, v)
        can_force(g, s, c, u, v)
    }
}

/// One round of forcing is monotone in the blue set.
///
/// Starting from more blue vertices never reaches fewer. This is not automatic: colouring a
/// vertex blue removes it from a forcing vertex's white neighbors, so a force that was
/// blocked by two white neighbors can become available. It never goes the other way.
theorem derived_set_mono[V](
    g: SimpleGraph[V], s: FiniteSet[V], b: FiniteSet[V], c: FiniteSet[V]
) {
    b.subset_eq(c) and c.subset_eq(s) implies derived_set(g, s, b).subset_eq(derived_set(g, s, c))
} by {
    if b.subset_eq(c) and c.subset_eq(s) {
        forall(v: V) {
            if derived_set(g, s, b).contains(v) {
                derived_set_contains_eq(g, s, b, v)
                s.contains(v)
                if c.contains(v) {
                    derived_set_contains_eq(g, s, c, v)
                    derived_set(g, s, c).contains(v)
                }
                if not c.contains(v) {
                    if b.contains(v) {
                        finite_set_subset_contains(b, c, v)
                        c.contains(v)
                        false
                    }
                    not b.contains(v)
                    is_forceable(g, s, b, v)
                    is_forceable_witness(g, s, b, v)
                    let (u: V) satisfy {
                        can_force(g, s, b, u, v)
                    }
                    can_force_of_subset(g, s, b, c, u, v)
                    can_force(g, s, c, u, v)
                    is_forceable_intro(g, s, c, u, v)
                    is_forceable(g, s, c, v)
                    derived_set_contains_eq(g, s, c, v)
                    derived_set(g, s, c).contains(v)
                }
                derived_set(g, s, c).contains(v)
            }
            (derived_set(g, s, b).contains(v) implies derived_set(g, s, c).contains(v))
        }
        fs_subset_eq_intro(derived_set(g, s, b), derived_set(g, s, c))
        derived_set(g, s, b).subset_eq(derived_set(g, s, c))
    }
}

/// Every round of forcing is monotone in the starting set.
theorem forcing_iterate_mono[V](
    g: SimpleGraph[V], s: FiniteSet[V], b: FiniteSet[V], c: FiniteSet[V], n: Nat
) {
    b.subset_eq(c) and c.subset_eq(s)
        implies forcing_iterate(g, s, b, n).subset_eq(forcing_iterate(g, s, c, n))
} by {
    if b.subset_eq(c) and c.subset_eq(s) {
        define p(x: Nat) -> Bool {
            forcing_iterate(g, s, b, x).subset_eq(forcing_iterate(g, s, c, x))
        }
        forcing_iterate(g, s, b, Nat.0) = b
        forcing_iterate(g, s, c, Nat.0) = c
        forcing_iterate(g, s, b, Nat.0).subset_eq(forcing_iterate(g, s, c, Nat.0))
        p(Nat.0)
        forall(k: Nat) {
            if p(k) {
                forcing_iterate(g, s, b, k).subset_eq(forcing_iterate(g, s, c, k))
                forcing_iterate_subset(g, s, c, k)
                forcing_iterate(g, s, c, k).subset_eq(s)
                derived_set_mono(g, s, forcing_iterate(g, s, b, k), forcing_iterate(g, s, c, k))
                derived_set(g, s, forcing_iterate(g, s, b, k)).subset_eq(
                    derived_set(g, s, forcing_iterate(g, s, c, k)))
                forcing_iterate_suc(g, s, b, k)
                forcing_iterate_suc(g, s, c, k)
                forcing_iterate(g, s, b, k.suc).subset_eq(forcing_iterate(g, s, c, k.suc))
                p(k.suc)
            }
            (p(k) implies p(k.suc))
        }
        p(Nat.0) and forall(k: Nat) {
            p(k) implies p(k.suc)
        }
        Nat.induction(p)
        p(n)
        forcing_iterate(g, s, b, n).subset_eq(forcing_iterate(g, s, c, n))
    }
}

/// A superset of a zero forcing set is a zero forcing set.
///
/// The larger start reaches at least as much, and it cannot reach more than the whole vertex
/// set, so it reaches exactly that.
theorem zero_forcing_set_of_superset[V](
    g: SimpleGraph[V], s: FiniteSet[V], b: FiniteSet[V], c: FiniteSet[V]
) {
    is_zero_forcing_set(g, s, b) and b.subset_eq(c) and c.subset_eq(s)
        implies is_zero_forcing_set(g, s, c)
} by {
    if is_zero_forcing_set(g, s, b) and b.subset_eq(c) and c.subset_eq(s) {
        is_zero_forcing_set_apply(g, s, b)
        exists(m: Nat) {
            forcing_iterate(g, s, b, m) = s
        }
        let (n: Nat) satisfy {
            forcing_iterate(g, s, b, n) = s
        }
        forcing_iterate_mono(g, s, b, c, n)
        forcing_iterate(g, s, b, n).subset_eq(forcing_iterate(g, s, c, n))
        s.subset_eq(forcing_iterate(g, s, c, n))
        forcing_iterate_subset(g, s, c, n)
        forcing_iterate(g, s, c, n).subset_eq(s)
        finite_set_subset_antisymm(forcing_iterate(g, s, c, n), s)
        forcing_iterate(g, s, c, n) = s
        is_zero_forcing_set_intro(g, s, c, n)
        is_zero_forcing_set(g, s, c)
    }
}
