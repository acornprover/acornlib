from nat import Nat
from pair import Pair
from finite_set import FiniteSet, fs_union, fs_insert, finite_set_ext_contains,
    finite_set_union_contains_eq, finite_set_empty_contains_eq
from data.finite.finite_set_card import fs_card, fs_card_empty
from graph.simple_graph import SimpleGraph
from graph.simple_graph_degree_sum import directed_edges_from
from graph.simple_graph_edge_fibers import directed_edges_from_set,
    directed_edges_from_set_contains_eq, directed_edges_from_set_insert_forward,
    directed_edges_from_set_insert_backward

numerals Nat

/// Enlarging the index set splits the edges into a union.
///
/// The two membership implications assembled into a set equality, which is the form the
/// counting step consumes.
theorem directed_edges_from_set_insert[V](
    g: SimpleGraph[V], s: FiniteSet[V], t: FiniteSet[V], v: V
) {
    directed_edges_from_set(g, s, fs_insert(t, v)) = fs_union(directed_edges_from(g, s, v), directed_edges_from_set(g, s, t))
} by {
    forall(p: Pair[V, V]) {
        finite_set_union_contains_eq(directed_edges_from(g, s, v), directed_edges_from_set(g, s, t), p)
        fs_union(directed_edges_from(g, s, v), directed_edges_from_set(g, s, t)).contains(p) = (directed_edges_from(g, s, v).contains(p) or directed_edges_from_set(g, s, t).contains(p))
        if directed_edges_from_set(g, s, fs_insert(t, v)).contains(p) {
            directed_edges_from_set_insert_forward(g, s, t, v, p)
            directed_edges_from(g, s, v).contains(p) or directed_edges_from_set(g, s, t).contains(p)
            fs_union(directed_edges_from(g, s, v), directed_edges_from_set(g, s, t)).contains(p)
        }
        if fs_union(directed_edges_from(g, s, v), directed_edges_from_set(g, s, t)).contains(p) {
            directed_edges_from(g, s, v).contains(p) or directed_edges_from_set(g, s, t).contains(p)
            directed_edges_from_set_insert_backward(g, s, t, v, p)
            directed_edges_from_set(g, s, fs_insert(t, v)).contains(p)
        }
        directed_edges_from_set(g, s, fs_insert(t, v)).contains(p) = fs_union(directed_edges_from(g, s, v), directed_edges_from_set(g, s, t)).contains(p)
        directed_edges_from_set(g, s, fs_insert(t, v)).contains(p) = directed_edges_from_set(g, s, fs_insert(t, v)).underlying_set.contains(p)
        fs_union(directed_edges_from(g, s, v), directed_edges_from_set(g, s, t)).contains(p) = fs_union(directed_edges_from(g, s, v), directed_edges_from_set(g, s, t)).underlying_set.contains(p)
        directed_edges_from_set(g, s, fs_insert(t, v)).underlying_set.contains(p) = fs_union(directed_edges_from(g, s, v), directed_edges_from_set(g, s, t)).underlying_set.contains(p)
    }
    finite_set_ext_contains(directed_edges_from_set(g, s, fs_insert(t, v)), fs_union(directed_edges_from(g, s, v), directed_edges_from_set(g, s, t)))
}

/// No edge leaves the empty index set.
theorem directed_edges_from_set_empty[V](g: SimpleGraph[V], s: FiniteSet[V]) {
    directed_edges_from_set(g, s, FiniteSet.empty[V]) = FiniteSet.empty[Pair[V, V]]
} by {
    forall(p: Pair[V, V]) {
        directed_edges_from_set_contains_eq(g, s, FiniteSet.empty[V], p)
        finite_set_empty_contains_eq(p.first)
        not FiniteSet.empty[V].contains(p.first)
        not directed_edges_from_set(g, s, FiniteSet.empty[V]).contains(p)
        finite_set_empty_contains_eq[Pair[V, V]](p)
        not FiniteSet.empty[Pair[V, V]].contains(p)
        directed_edges_from_set(g, s, FiniteSet.empty[V]).contains(p) = FiniteSet.empty[Pair[V, V]].contains(p)
        directed_edges_from_set(g, s, FiniteSet.empty[V]).contains(p) = directed_edges_from_set(g, s, FiniteSet.empty[V]).underlying_set.contains(p)
        FiniteSet.empty[Pair[V, V]].contains(p) = FiniteSet.empty[Pair[V, V]].underlying_set.contains(p)
        directed_edges_from_set(g, s, FiniteSet.empty[V]).underlying_set.contains(p) = FiniteSet.empty[Pair[V, V]].underlying_set.contains(p)
    }
    finite_set_ext_contains(directed_edges_from_set(g, s, FiniteSet.empty[V]), FiniteSet.empty[Pair[V, V]])
}

/// No edge leaves the empty index set, counted.
theorem directed_edges_from_set_empty_card[V](g: SimpleGraph[V], s: FiniteSet[V]) {
    fs_card(directed_edges_from_set(g, s, FiniteSet.empty[V])) = Nat.0
} by {
    directed_edges_from_set_empty(g, s)
    directed_edges_from_set(g, s, FiniteSet.empty[V]) = FiniteSet.empty[Pair[V, V]]
    fs_card_empty[Pair[V, V]]
    fs_card(FiniteSet.empty[Pair[V, V]]) = Nat.0
}
