from nat import Nat
from finite_set import FiniteSet
from data.nat.nat_range_set import range_set, range_set_contains
from graph.simple_graph import complete_graph, complete_graph_adj_iff_ne
from graph.simple_graph_degree import neighborhood, neighborhood_contains_eq
from graph.ramsey34_labels import nine_labels_lt_ladder, nine_labels_lt_bound

numerals Nat

/// The nine vertices `{0, ..., 8}` on which the upper bound R(3, 4) <= 9 lives.
let nine_vertices: FiniteSet[Nat] = range_set(Nat.9)

/// The eight vertices `{0, ..., 7}` on which the lower bound R(3, 4) > 8 lives.
let eight_vertices: FiniteSet[Nat] = range_set(Nat.8)

/// A natural below nine is a vertex of the nine-element set.
theorem nine_vertices_contains(x: Nat) {
    x < Nat.9 implies nine_vertices.contains(x)
} by {
    if x < Nat.9 {
        range_set_contains(Nat.9, x)
        range_set(Nat.9).contains(x)
        nine_vertices.contains(x)
    }
}

/// A natural below eight is a vertex of the eight-element set.
theorem eight_vertices_contains(x: Nat) {
    x < Nat.8 implies eight_vertices.contains(x)
} by {
    if x < Nat.8 {
        range_set_contains(Nat.8, x)
        range_set(Nat.8).contains(x)
        eight_vertices.contains(x)
    }
}

/// A vertex of the nine-element set other than zero is a neighbour of zero in the complete graph.
theorem nine_neighbor_of_zero(x: Nat) {
    nine_vertices.contains(x) and x != Nat.0 implies
        neighborhood(complete_graph[Nat], nine_vertices, Nat.0).contains(x)
} by {
    if nine_vertices.contains(x) and x != Nat.0 {
        neighborhood_contains_eq(complete_graph[Nat], nine_vertices, Nat.0, x)
        (neighborhood(complete_graph[Nat], nine_vertices, Nat.0).contains(x)
            = (nine_vertices.contains(x) and complete_graph[Nat].adj(Nat.0, x)))
        complete_graph_adj_iff_ne[Nat](Nat.0, x)
        complete_graph[Nat].adj(Nat.0, x) = (Nat.0 != x)
        nine_vertices.contains(x)
        complete_graph[Nat].adj(Nat.0, x)
        neighborhood(complete_graph[Nat], nine_vertices, Nat.0).contains(x)
    }
}
