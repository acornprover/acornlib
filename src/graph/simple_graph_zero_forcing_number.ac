from nat import Nat, is_min, has_min, is_min_apply, is_min_false_below, false_below,
    false_below_apply, lte_antisymm
from finite_set import FiniteSet, finite_set_subset_refl
from data.finite.finite_set_card import fs_card
from graph.simple_graph import SimpleGraph
from graph.simple_graph_zero_forcing_closure import is_zero_forcing_set, is_zero_forcing_set_apply,
    ambient_is_zero_forcing_set

numerals Nat

/// True of the sizes achieved by zero forcing subsets of `s`.
///
/// Packaged as a predicate on `Nat` so the minimum construction applies to it.
define zero_forcing_size_pred[V](
    g: SimpleGraph[V], s: FiniteSet[V]
) -> (Nat -> Bool) {
    function(n: Nat) {
        exists(b: FiniteSet[V]) {
            is_zero_forcing_set(g, s, b) and fs_card(b) = n
        }
    }
}

/// A zero forcing set achieves its own size.
theorem zero_forcing_size_pred_intro[V](
    g: SimpleGraph[V], s: FiniteSet[V], b: FiniteSet[V]
) {
    is_zero_forcing_set(g, s, b) implies zero_forcing_size_pred(g, s)(fs_card(b))
} by {
    if is_zero_forcing_set(g, s, b) {
        zero_forcing_size_pred(g, s)(fs_card(b)) = exists(c: FiniteSet[V]) {
            is_zero_forcing_set(g, s, c) and fs_card(c) = fs_card(b)
        }
        exists(c: FiniteSet[V]) {
            is_zero_forcing_set(g, s, c) and fs_card(c) = fs_card(b)
        }
        zero_forcing_size_pred(g, s)(fs_card(b))
    }
}

/// The ambient set itself achieves a zero forcing size.
theorem zero_forcing_size_pred_ambient[V](g: SimpleGraph[V], s: FiniteSet[V]) {
    zero_forcing_size_pred(g, s)(fs_card(s))
} by {
    ambient_is_zero_forcing_set(g, s)
    is_zero_forcing_set(g, s, s)
    zero_forcing_size_pred_intro(g, s, s)
    zero_forcing_size_pred(g, s)(fs_card(s))
}

/// The fewest vertices in a zero forcing subset of `s`.
///
/// Well defined because the whole vertex set is a zero forcing set, being already entirely
/// blue, so some zero forcing size exists and every nonempty set of naturals has a least
/// element. Mathlib has no counterpart to this invariant.
let zero_forcing_number[V](g: SimpleGraph[V], s: FiniteSet[V]) -> result: Nat satisfy {
    is_min(zero_forcing_size_pred(g, s), result)
} by {
    zero_forcing_size_pred_ambient(g, s)
    has_min(zero_forcing_size_pred(g, s), fs_card(s))
}

/// Some zero forcing set attains the zero forcing number.
theorem zero_forcing_number_attained[V](g: SimpleGraph[V], s: FiniteSet[V]) {
    zero_forcing_size_pred(g, s)(zero_forcing_number(g, s))
} by {
    is_min(zero_forcing_size_pred(g, s), zero_forcing_number(g, s))
    is_min_apply(zero_forcing_size_pred(g, s), zero_forcing_number(g, s))
}

/// No zero forcing set is smaller than the zero forcing number.
theorem zero_forcing_number_is_least[V](
    g: SimpleGraph[V], s: FiniteSet[V], b: FiniteSet[V]
) {
    is_zero_forcing_set(g, s, b) implies zero_forcing_number(g, s) <= fs_card(b)
} by {
    if is_zero_forcing_set(g, s, b) {
        zero_forcing_size_pred_intro(g, s, b)
        zero_forcing_size_pred(g, s)(fs_card(b))
        is_min(zero_forcing_size_pred(g, s), zero_forcing_number(g, s))
        is_min_false_below(zero_forcing_size_pred(g, s), zero_forcing_number(g, s))
        false_below(zero_forcing_size_pred(g, s), zero_forcing_number(g, s))
        if fs_card(b) < zero_forcing_number(g, s) {
            false_below_apply(zero_forcing_size_pred(g, s), zero_forcing_number(g, s),
                fs_card(b))
            not zero_forcing_size_pred(g, s)(fs_card(b))
            false
        }
        zero_forcing_number(g, s) <= fs_card(b)
    }
}

/// The zero forcing number is at most the number of vertices.
theorem zero_forcing_number_le_ambient[V](g: SimpleGraph[V], s: FiniteSet[V]) {
    zero_forcing_number(g, s) <= fs_card(s)
} by {
    ambient_is_zero_forcing_set(g, s)
    is_zero_forcing_set(g, s, s)
    zero_forcing_number_is_least(g, s, s)
    zero_forcing_number(g, s) <= fs_card(s)
}

/// True if `b` is a zero forcing set of the fewest possible size.
define is_minimum_zero_forcing_set[V](
    g: SimpleGraph[V], s: FiniteSet[V], b: FiniteSet[V]
) -> Bool {
    is_zero_forcing_set(g, s, b) and fs_card(b) = zero_forcing_number(g, s)
}

/// A minimum zero forcing set is a zero forcing set of the stated size.
theorem is_minimum_zero_forcing_set_apply[V](
    g: SimpleGraph[V], s: FiniteSet[V], b: FiniteSet[V]
) {
    is_minimum_zero_forcing_set(g, s, b)
        implies is_zero_forcing_set(g, s, b) and fs_card(b) = zero_forcing_number(g, s)
} by {
    if is_minimum_zero_forcing_set(g, s, b) {
        is_minimum_zero_forcing_set(g, s, b) =
            (is_zero_forcing_set(g, s, b) and fs_card(b) = zero_forcing_number(g, s))
        is_zero_forcing_set(g, s, b) and fs_card(b) = zero_forcing_number(g, s)
    }
}

/// The two conditions give a minimum zero forcing set.
theorem is_minimum_zero_forcing_set_intro[V](
    g: SimpleGraph[V], s: FiniteSet[V], b: FiniteSet[V]
) {
    is_zero_forcing_set(g, s, b) and fs_card(b) = zero_forcing_number(g, s)
        implies is_minimum_zero_forcing_set(g, s, b)
} by {
    if is_zero_forcing_set(g, s, b) and fs_card(b) = zero_forcing_number(g, s) {
        is_minimum_zero_forcing_set(g, s, b) =
            (is_zero_forcing_set(g, s, b) and fs_card(b) = zero_forcing_number(g, s))
        is_minimum_zero_forcing_set(g, s, b)
    }
}

/// A minimum zero forcing set exists.
theorem minimum_zero_forcing_set_exists[V](g: SimpleGraph[V], s: FiniteSet[V]) {
    exists(b: FiniteSet[V]) { is_minimum_zero_forcing_set(g, s, b) }
} by {
    zero_forcing_number_attained(g, s)
    zero_forcing_size_pred(g, s)(zero_forcing_number(g, s))
    zero_forcing_size_pred(g, s)(zero_forcing_number(g, s)) = exists(c: FiniteSet[V]) {
        is_zero_forcing_set(g, s, c) and fs_card(c) = zero_forcing_number(g, s)
    }
    exists(c: FiniteSet[V]) {
        is_zero_forcing_set(g, s, c) and fs_card(c) = zero_forcing_number(g, s)
    }
    let (b: FiniteSet[V]) satisfy {
        is_zero_forcing_set(g, s, b) and fs_card(b) = zero_forcing_number(g, s)
    }
    is_minimum_zero_forcing_set_intro(g, s, b)
    is_minimum_zero_forcing_set(g, s, b)
    exists(c: FiniteSet[V]) { is_minimum_zero_forcing_set(g, s, c) }
}

/// A zero forcing set no larger than a minimum one is itself minimum.
theorem is_minimum_zero_forcing_set_of_lte[V](
    g: SimpleGraph[V], s: FiniteSet[V], b: FiniteSet[V], c: FiniteSet[V]
) {
    is_minimum_zero_forcing_set(g, s, c) and is_zero_forcing_set(g, s, b)
        and fs_card(b) <= fs_card(c)
        implies is_minimum_zero_forcing_set(g, s, b)
} by {
    if is_minimum_zero_forcing_set(g, s, c) and is_zero_forcing_set(g, s, b)
        and fs_card(b) <= fs_card(c) {
        is_minimum_zero_forcing_set_apply(g, s, c)
        fs_card(c) = zero_forcing_number(g, s)
        fs_card(b) <= zero_forcing_number(g, s)
        zero_forcing_number_is_least(g, s, b)
        zero_forcing_number(g, s) <= fs_card(b)
        lte_antisymm(fs_card(b), zero_forcing_number(g, s))
        fs_card(b) = zero_forcing_number(g, s)
        is_minimum_zero_forcing_set_intro(g, s, b)
        is_minimum_zero_forcing_set(g, s, b)
    }
}

/// Any two minimum zero forcing sets have the same size.
theorem minimum_zero_forcing_sets_same_size[V](
    g: SimpleGraph[V], s: FiniteSet[V], b: FiniteSet[V], c: FiniteSet[V]
) {
    is_minimum_zero_forcing_set(g, s, b) and is_minimum_zero_forcing_set(g, s, c)
        implies fs_card(b) = fs_card(c)
} by {
    if is_minimum_zero_forcing_set(g, s, b) and is_minimum_zero_forcing_set(g, s, c) {
        is_minimum_zero_forcing_set_apply(g, s, b)
        fs_card(b) = zero_forcing_number(g, s)
        is_minimum_zero_forcing_set_apply(g, s, c)
        fs_card(c) = zero_forcing_number(g, s)
        fs_card(b) = fs_card(c)
    }
}
