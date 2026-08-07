from nat import Nat
from pair import Pair
from finite_set import FiniteSet
from finite_set import finite_set_sum
from data.finite.finite_set_filter import finite_set_filter, finite_set_filter_contains_eq,
    finite_set_filter_subset
from data.finite.finite_set_card import fs_card
from data.finite.finite_set_card_compare import fs_card_mono
from graph.simple_graph import SimpleGraph
from graph.simple_graph_degree import degree, neighborhood, neighborhood_contains_eq
from graph.simple_graph_edges import directed_edges, directed_edges_contains_eq,
    directed_edges_contains_pair, directed_edge_count

numerals Nat

/// The degree function of `g` on `s`, as a map from vertices to counts.
///
/// Named so it can be summed; `finite_set_sum` takes a function, not an expression.
define degree_fn[V](g: SimpleGraph[V], s: FiniteSet[V]) -> (V -> Nat) {
    function(v: V) {
        degree(g, s, v)
    }
}

/// The degree map agrees with `degree`.
theorem degree_fn_eq[V](g: SimpleGraph[V], s: FiniteSet[V], v: V) {
    degree_fn(g, s)(v) = degree(g, s, v)
}

/// The sum of the degrees of every vertex of `s`.
define degree_sum[V](g: SimpleGraph[V], s: FiniteSet[V]) -> Nat {
    finite_set_sum(s, degree_fn(g, s))
}

/// The degree sum is the sum of the degree map.
theorem degree_sum_eq[V](g: SimpleGraph[V], s: FiniteSet[V]) {
    degree_sum(g, s) = finite_set_sum(s, degree_fn(g, s))
}

/// True of the pairs whose first component is `v`.
define first_is[V](v: V) -> (Pair[V, V] -> Bool) {
    function(p: Pair[V, V]) {
        p.first = v
    }
}

/// A pair has first component `v` exactly when the predicate holds.
theorem first_is_eq[V](v: V, p: Pair[V, V]) {
    first_is(v)(p) = (p.first = v)
}

/// The directed edges leaving `v`.
define directed_edges_from[V](
    g: SimpleGraph[V], s: FiniteSet[V], v: V
) -> FiniteSet[Pair[V, V]] {
    finite_set_filter(directed_edges(g, s), first_is(v))
}

/// A pair leaves `v` exactly when it is a directed edge with first component `v`.
theorem directed_edges_from_contains_eq[V](
    g: SimpleGraph[V], s: FiniteSet[V], v: V, p: Pair[V, V]
) {
    directed_edges_from(g, s, v).contains(p) = (directed_edges(g, s).contains(p) and p.first = v)
} by {
    finite_set_filter_contains_eq(directed_edges(g, s), first_is(v), p)
    directed_edges_from(g, s, v).contains(p) = (directed_edges(g, s).contains(p) and first_is(v)(p))
    first_is_eq(v, p)
    first_is(v)(p) = (p.first = v)
}

/// The edges leaving `v` are exactly the pairs from `v` to its neighbors.
theorem directed_edges_from_contains_pair[V](
    g: SimpleGraph[V], s: FiniteSet[V], v: V, w: V
) {
    s.contains(v) and neighborhood(g, s, v).contains(w) implies directed_edges_from(g, s, v).contains(Pair.new(v, w))
} by {
    if s.contains(v) and neighborhood(g, s, v).contains(w) {
        neighborhood_contains_eq(g, s, v, w)
        s.contains(w) and g.adj(v, w)
        directed_edges_contains_pair(g, s, v, w)
        directed_edges(g, s).contains(Pair.new(v, w))
        Pair.new(v, w).first = v
        directed_edges_from_contains_eq(g, s, v, Pair.new(v, w))
        directed_edges_from(g, s, v).contains(Pair.new(v, w))
    }
}

/// A directed edge leaving `v` ends at a neighbor of `v`.
theorem directed_edges_from_second_is_neighbor[V](
    g: SimpleGraph[V], s: FiniteSet[V], v: V, p: Pair[V, V]
) {
    directed_edges_from(g, s, v).contains(p) implies neighborhood(g, s, v).contains(p.second)
} by {
    if directed_edges_from(g, s, v).contains(p) {
        directed_edges_from_contains_eq(g, s, v, p)
        directed_edges(g, s).contains(p)
        p.first = v
        directed_edges_contains_eq(g, s, p)
        s.contains(p.first) and s.contains(p.second) and g.adj(p.first, p.second)
        s.contains(p.second)
        g.adj(v, p.second)
        neighborhood_contains_eq(g, s, v, p.second)
        neighborhood(g, s, v).contains(p.second)
    }
}

/// Edges leaving a vertex are directed edges.
theorem directed_edges_from_subset[V](g: SimpleGraph[V], s: FiniteSet[V], v: V) {
    directed_edges_from(g, s, v).subset_eq(directed_edges(g, s))
} by {
    finite_set_filter_subset(directed_edges(g, s), first_is(v))
}

/// The edges leaving one vertex are no more numerous than all directed edges.
theorem directed_edges_from_card_le[V](g: SimpleGraph[V], s: FiniteSet[V], v: V) {
    fs_card(directed_edges_from(g, s, v)) <= directed_edge_count(g, s)
} by {
    directed_edges_from_subset(g, s, v)
    directed_edges_from(g, s, v).subset_eq(directed_edges(g, s))
    fs_card_mono(directed_edges_from(g, s, v), directed_edges(g, s))
    fs_card(directed_edges_from(g, s, v)) <= fs_card(directed_edges(g, s))
    directed_edge_count(g, s) = fs_card(directed_edges(g, s))
}
