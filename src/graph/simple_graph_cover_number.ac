from nat import Nat, lte_trans, is_min, has_min, is_min_apply, is_min_false_below, false_below,
    false_below_apply
from finite_set import FiniteSet, finite_set_subset_refl
from data.finite.finite_set_card import fs_card
from graph.simple_graph import SimpleGraph
from graph.simple_graph_vertex_sets import is_vertex_cover, is_vertex_cover_self,
    is_vertex_cover_of_subset

numerals Nat

/// True of the sizes achieved by vertex covers of `s`.
define cover_size_pred[V](g: SimpleGraph[V], s: FiniteSet[V]) -> (Nat -> Bool) {
    function(n: Nat) {
        exists(c: FiniteSet[V]) {
            c.subset_eq(s) and is_vertex_cover(g, s, c) and fs_card(c) = n
        }
    }
}

/// A vertex cover achieves its own size.
theorem cover_size_pred_intro[V](
    g: SimpleGraph[V], s: FiniteSet[V], c: FiniteSet[V]
) {
    c.subset_eq(s) and is_vertex_cover(g, s, c) implies cover_size_pred(g, s)(fs_card(c))
} by {
    if c.subset_eq(s) and is_vertex_cover(g, s, c) {
        cover_size_pred(g, s)(fs_card(c)) = exists(e: FiniteSet[V]) {
            e.subset_eq(s) and is_vertex_cover(g, s, e) and fs_card(e) = fs_card(c)
        }
        exists(e: FiniteSet[V]) {
            e.subset_eq(s) and is_vertex_cover(g, s, e) and fs_card(e) = fs_card(c)
        }
        cover_size_pred(g, s)(fs_card(c))
    }
}

/// The ambient set itself achieves a cover size.
theorem cover_size_pred_ambient[V](g: SimpleGraph[V], s: FiniteSet[V]) {
    cover_size_pred(g, s)(fs_card(s))
} by {
    finite_set_subset_refl(s)
    s.subset_eq(s)
    is_vertex_cover_self(g, s)
    is_vertex_cover(g, s, s)
    cover_size_pred_intro(g, s, s)
    cover_size_pred(g, s)(fs_card(s))
}

/// The fewest vertices in a vertex cover of `s`.
///
/// Well defined because the ambient set covers itself, so some cover size exists,
/// and every nonempty set of naturals has a least element.
let vertex_cover_number[V](g: SimpleGraph[V], s: FiniteSet[V]) -> result: Nat satisfy {
    is_min(cover_size_pred(g, s), result)
} by {
    cover_size_pred_ambient(g, s)
    has_min(cover_size_pred(g, s), fs_card(s))
}

/// Some vertex cover attains the cover number.
theorem vertex_cover_number_attained[V](g: SimpleGraph[V], s: FiniteSet[V]) {
    cover_size_pred(g, s)(vertex_cover_number(g, s))
} by {
    is_min(cover_size_pred(g, s), vertex_cover_number(g, s))
    is_min_apply(cover_size_pred(g, s), vertex_cover_number(g, s))
}

/// No vertex cover is smaller than the cover number.
theorem vertex_cover_number_is_least[V](
    g: SimpleGraph[V], s: FiniteSet[V], c: FiniteSet[V]
) {
    c.subset_eq(s) and is_vertex_cover(g, s, c) implies vertex_cover_number(g, s) <= fs_card(c)
} by {
    if c.subset_eq(s) and is_vertex_cover(g, s, c) {
        cover_size_pred_intro(g, s, c)
        cover_size_pred(g, s)(fs_card(c))
        is_min(cover_size_pred(g, s), vertex_cover_number(g, s))
        is_min_false_below(cover_size_pred(g, s), vertex_cover_number(g, s))
        false_below(cover_size_pred(g, s), vertex_cover_number(g, s))
        if fs_card(c) < vertex_cover_number(g, s) {
            false_below_apply(cover_size_pred(g, s), vertex_cover_number(g, s), fs_card(c))
            not cover_size_pred(g, s)(fs_card(c))
            false
        }
        vertex_cover_number(g, s) <= fs_card(c)
    }
}

/// The cover number is at most the number of vertices.
theorem vertex_cover_number_le_ambient[V](g: SimpleGraph[V], s: FiniteSet[V]) {
    vertex_cover_number(g, s) <= fs_card(s)
} by {
    finite_set_subset_refl(s)
    s.subset_eq(s)
    is_vertex_cover_self(g, s)
    is_vertex_cover(g, s, s)
    vertex_cover_number_is_least(g, s, s)
    vertex_cover_number(g, s) <= fs_card(s)
}

/// A cover size bound gives a cover-number bound.
theorem vertex_cover_number_le_of_cover[V](
    g: SimpleGraph[V], s: FiniteSet[V], c: FiniteSet[V], n: Nat
) {
    c.subset_eq(s) and is_vertex_cover(g, s, c) and fs_card(c) <= n
        implies vertex_cover_number(g, s) <= n
} by {
    if c.subset_eq(s) and is_vertex_cover(g, s, c) and fs_card(c) <= n {
        vertex_cover_number_is_least(g, s, c)
        vertex_cover_number(g, s) <= fs_card(c)
        lte_trans(vertex_cover_number(g, s), fs_card(c), n)
        vertex_cover_number(g, s) <= n
    }
}
