from data.basic.relation_basic import is_symmetric, is_irreflexive
from data.basic.functions import binary_function_extensionality, compose, compose_injective_fn, identity_fn, is_injective_fn, is_left_inverse_fn, left_inverse_fn_imp_injective_fn
from data.basic.set import Set, set_image, set_preimage, maps_into_set_image, set_preimage_contains_eq, set_image_contains_eq, image_contains, subset_contains

/// A simple graph on a vertex type `V`, represented by an irreflexive symmetric adjacency relation.
/// Edges are unordered and there are no self-loops.
structure SimpleGraph[V] {
    /// True if the two vertices are adjacent.
    adj: (V, V) -> Bool
} constraint {
    is_symmetric(adj) and is_irreflexive(adj)
}

/// The adjacency relation of a simple graph is symmetric.
theorem simple_graph_adj_symmetric[V](g: SimpleGraph[V], x: V, y: V) {
    g.adj(x, y) implies g.adj(y, x)
} by {
    is_symmetric(g.adj)
}

/// No vertex is adjacent to itself.
theorem simple_graph_adj_irreflexive[V](g: SimpleGraph[V], x: V) {
    not g.adj(x, x)
} by {
    is_irreflexive(g.adj)
}

/// Symmetric form of adjacency.
theorem simple_graph_adj_comm[V](g: SimpleGraph[V], x: V, y: V) {
    g.adj(x, y) = g.adj(y, x)
} by {
    if g.adj(x, y) {
        simple_graph_adj_symmetric(g, x, y)
        g.adj(y, x)
    }
    if g.adj(y, x) {
        simple_graph_adj_symmetric(g, y, x)
        g.adj(x, y)
    }
}

/// Adjacent vertices are distinct.
theorem simple_graph_adj_ne[V](g: SimpleGraph[V], x: V, y: V) {
    g.adj(x, y) implies x != y
} by {
    if g.adj(x, y) {
        if x = y {
            g.adj(x, x)
            simple_graph_adj_irreflexive(g, x)
            false
        }
        x != y
    }
}

/// Two simple graphs are equal when their adjacency relations agree pointwise.
theorem simple_graph_ext[V](g: SimpleGraph[V], h: SimpleGraph[V]) {
    (forall(x: V, y: V) { g.adj(x, y) = h.adj(x, y) }) implies g = h
} by {
    if forall(x: V, y: V) { g.adj(x, y) = h.adj(x, y) } {
        binary_function_extensionality(g.adj, h.adj)
        g.adj = h.adj
    }
}

/// Equal simple graphs have equal adjacency relations.
theorem simple_graph_eq_adj[V](g: SimpleGraph[V], h: SimpleGraph[V]) {
    g = h implies g.adj = h.adj
}

/// The empty graph on `V`: no two vertices are adjacent.
define empty_graph_adj[V](x: V, y: V) -> Bool {
    false
}

/// `empty_graph_adj` satisfies the simple-graph constraint.
theorem empty_graph_is_symmetric[V] {
    is_symmetric(empty_graph_adj[V])
}

/// `empty_graph_adj` has no loops.
theorem empty_graph_is_irreflexive[V] {
    is_irreflexive(empty_graph_adj[V])
}

/// `empty_graph_adj` satisfies the simple-graph constraint.
theorem empty_graph_constraint[V] {
    is_symmetric(empty_graph_adj[V]) and is_irreflexive(empty_graph_adj[V])
} by {
    empty_graph_is_symmetric[V]
    empty_graph_is_irreflexive[V]
}

/// The empty graph on `V`: no two vertices are adjacent.
let empty_graph[V]: SimpleGraph[V] satisfy {
    SimpleGraph.new(empty_graph_adj[V]) = Option.some(empty_graph)
}

/// In the empty graph, no two vertices are adjacent.
theorem empty_graph_adj_false[V](x: V, y: V) {
    not empty_graph[V].adj(x, y)
}

/// The complete graph on `V`: distinct vertices are always adjacent.
define complete_graph_adj[V](x: V, y: V) -> Bool {
    x != y
}

/// `complete_graph_adj` satisfies the simple-graph constraint.
theorem complete_graph_is_symmetric[V] {
    is_symmetric(complete_graph_adj[V])
} by {
    forall(x: V, y: V) {
        if complete_graph_adj[V](x, y) {
            x != y
            y != x
            complete_graph_adj[V](y, x)
        }
    }
}

/// `complete_graph_adj` has no loops.
theorem complete_graph_is_irreflexive[V] {
    is_irreflexive(complete_graph_adj[V])
}

/// `complete_graph_adj` satisfies the simple-graph constraint.
theorem complete_graph_constraint[V] {
    is_symmetric(complete_graph_adj[V]) and is_irreflexive(complete_graph_adj[V])
} by {
    complete_graph_is_symmetric[V]
    complete_graph_is_irreflexive[V]
}

/// The complete graph on `V`: distinct vertices are always adjacent.
let complete_graph[V]: SimpleGraph[V] satisfy {
    SimpleGraph.new(complete_graph_adj[V]) = Option.some(complete_graph)
}

/// In the complete graph, distinct vertices are adjacent.
theorem complete_graph_adj_iff_ne[V](x: V, y: V) {
    complete_graph[V].adj(x, y) = (x != y)
}

/// The adjacency relation of the complement graph.
define graph_complement_adj[V](g: SimpleGraph[V], x: V, y: V) -> Bool {
    x != y and not g.adj(x, y)
}

/// The complement adjacency relation is symmetric.
theorem graph_complement_adj_is_symmetric[V](g: SimpleGraph[V]) {
    is_symmetric(graph_complement_adj(g))
} by {
    forall(x: V, y: V) {
        if graph_complement_adj(g, x, y) {
            graph_complement_adj(g, x, y) = (x != y and not g.adj(x, y))
            x != y
            y != x
            not g.adj(x, y)
            simple_graph_adj_comm(g, x, y)
            if g.adj(y, x) {
                g.adj(x, y)
                false
            }
            not g.adj(y, x)
            graph_complement_adj(g, y, x) = (y != x and not g.adj(y, x))
            graph_complement_adj(g, y, x)
        }
    }
}

/// The complement adjacency relation has no loops.
theorem graph_complement_adj_is_irreflexive[V](g: SimpleGraph[V]) {
    is_irreflexive(graph_complement_adj(g))
} by {
    forall(x: V) {
        if graph_complement_adj(g, x, x) {
            graph_complement_adj(g, x, x) = (x != x and not g.adj(x, x))
            false
        }
    }
}

/// The complement adjacency relation satisfies the simple-graph constraint.
theorem graph_complement_adj_constraint[V](g: SimpleGraph[V]) {
    is_symmetric(graph_complement_adj(g)) and is_irreflexive(graph_complement_adj(g))
} by {
    graph_complement_adj_is_symmetric(g)
    graph_complement_adj_is_irreflexive(g)
}

/// The complement of a simple graph on the same vertices.
let graph_complement[V](g: SimpleGraph[V]) -> result: SimpleGraph[V] satisfy {
    SimpleGraph.new(graph_complement_adj(g)) = Option.some(result)
}

/// Adjacency in the complement is distinct non-adjacency in the original graph.
theorem graph_complement_adj_eq[V](g: SimpleGraph[V], x: V, y: V) {
    graph_complement(g).adj(x, y) = (x != y and not g.adj(x, y))
} by {
    graph_complement(g).adj = graph_complement_adj(g)
    graph_complement(g).adj(x, y) = graph_complement_adj(g, x, y)
    graph_complement_adj(g, x, y) = (x != y and not g.adj(x, y))
}

/// The adjacency relation of the union of two graphs on the same vertices.
define graph_union_adj[V](g: SimpleGraph[V], h: SimpleGraph[V], x: V, y: V) -> Bool {
    g.adj(x, y) or h.adj(x, y)
}

/// The union adjacency relation is symmetric.
theorem graph_union_adj_is_symmetric[V](g: SimpleGraph[V], h: SimpleGraph[V]) {
    is_symmetric(graph_union_adj(g, h))
} by {
    forall(x: V, y: V) {
        if graph_union_adj(g, h, x, y) {
            graph_union_adj(g, h, x, y) = (g.adj(x, y) or h.adj(x, y))
            if g.adj(x, y) {
                simple_graph_adj_symmetric(g, x, y)
                g.adj(y, x)
                graph_union_adj(g, h, y, x)
            } else {
                h.adj(x, y)
                simple_graph_adj_symmetric(h, x, y)
                h.adj(y, x)
                graph_union_adj(g, h, y, x)
            }
        }
    }
}

/// The union adjacency relation has no loops.
theorem graph_union_adj_is_irreflexive[V](g: SimpleGraph[V], h: SimpleGraph[V]) {
    is_irreflexive(graph_union_adj(g, h))
} by {
    forall(x: V) {
        if graph_union_adj(g, h, x, x) {
            graph_union_adj(g, h, x, x) = (g.adj(x, x) or h.adj(x, x))
            if g.adj(x, x) {
                simple_graph_adj_irreflexive(g, x)
                false
            } else {
                h.adj(x, x)
                simple_graph_adj_irreflexive(h, x)
                false
            }
        }
    }
}

/// The union adjacency relation satisfies the simple-graph constraint.
theorem graph_union_adj_constraint[V](g: SimpleGraph[V], h: SimpleGraph[V]) {
    is_symmetric(graph_union_adj(g, h)) and is_irreflexive(graph_union_adj(g, h))
} by {
    graph_union_adj_is_symmetric(g, h)
    graph_union_adj_is_irreflexive(g, h)
}

/// The union of two simple graphs on the same vertices.
let graph_union[V](g: SimpleGraph[V], h: SimpleGraph[V]) -> result: SimpleGraph[V] satisfy {
    SimpleGraph.new(graph_union_adj(g, h)) = Option.some(result)
}

/// Adjacency in the union is adjacency in at least one graph.
theorem graph_union_adj_eq[V](g: SimpleGraph[V], h: SimpleGraph[V], x: V, y: V) {
    graph_union(g, h).adj(x, y) = (g.adj(x, y) or h.adj(x, y))
} by {
    graph_union(g, h).adj = graph_union_adj(g, h)
    graph_union(g, h).adj(x, y) = graph_union_adj(g, h, x, y)
    graph_union_adj(g, h, x, y) = (g.adj(x, y) or h.adj(x, y))
}

/// The adjacency relation of the intersection of two graphs on the same vertices.
define graph_intersection_adj[V](g: SimpleGraph[V], h: SimpleGraph[V], x: V, y: V) -> Bool {
    g.adj(x, y) and h.adj(x, y)
}

/// The intersection adjacency relation is symmetric.
theorem graph_intersection_adj_is_symmetric[V](g: SimpleGraph[V], h: SimpleGraph[V]) {
    is_symmetric(graph_intersection_adj(g, h))
} by {
    forall(x: V, y: V) {
        if graph_intersection_adj(g, h, x, y) {
            graph_intersection_adj(g, h, x, y) = (g.adj(x, y) and h.adj(x, y))
            g.adj(x, y)
            h.adj(x, y)
            simple_graph_adj_symmetric(g, x, y)
            simple_graph_adj_symmetric(h, x, y)
            g.adj(y, x)
            h.adj(y, x)
            graph_intersection_adj(g, h, y, x)
        }
    }
}

/// The intersection adjacency relation has no loops.
theorem graph_intersection_adj_is_irreflexive[V](g: SimpleGraph[V], h: SimpleGraph[V]) {
    is_irreflexive(graph_intersection_adj(g, h))
} by {
    forall(x: V) {
        if graph_intersection_adj(g, h, x, x) {
            graph_intersection_adj(g, h, x, x) = (g.adj(x, x) and h.adj(x, x))
            g.adj(x, x)
            simple_graph_adj_irreflexive(g, x)
            false
        }
    }
}

/// The intersection adjacency relation satisfies the simple-graph constraint.
theorem graph_intersection_adj_constraint[V](g: SimpleGraph[V], h: SimpleGraph[V]) {
    is_symmetric(graph_intersection_adj(g, h)) and is_irreflexive(graph_intersection_adj(g, h))
} by {
    graph_intersection_adj_is_symmetric(g, h)
    graph_intersection_adj_is_irreflexive(g, h)
}

/// The intersection of two simple graphs on the same vertices.
let graph_intersection[V](g: SimpleGraph[V], h: SimpleGraph[V]) -> result: SimpleGraph[V] satisfy {
    SimpleGraph.new(graph_intersection_adj(g, h)) = Option.some(result)
}

/// Adjacency in the intersection is adjacency in both graphs.
theorem graph_intersection_adj_eq[V](g: SimpleGraph[V], h: SimpleGraph[V], x: V, y: V) {
    graph_intersection(g, h).adj(x, y) = (g.adj(x, y) and h.adj(x, y))
} by {
    graph_intersection(g, h).adj = graph_intersection_adj(g, h)
    graph_intersection(g, h).adj(x, y) = graph_intersection_adj(g, h, x, y)
    graph_intersection_adj(g, h, x, y) = (g.adj(x, y) and h.adj(x, y))
}

/// The complement of the empty graph is the complete graph.
theorem graph_complement_empty_graph[V] {
    graph_complement(empty_graph[V]) = complete_graph[V]
} by {
    forall(x: V, y: V) {
        graph_complement_adj_eq(empty_graph[V], x, y)
        complete_graph_adj_iff_ne[V](x, y)
        empty_graph_adj_false[V](x, y)
        if graph_complement(empty_graph[V]).adj(x, y) {
            x != y
            complete_graph[V].adj(x, y)
        }
        if complete_graph[V].adj(x, y) {
            x != y
            not empty_graph[V].adj(x, y)
            graph_complement(empty_graph[V]).adj(x, y)
        }
        graph_complement(empty_graph[V]).adj(x, y) = complete_graph[V].adj(x, y)
    }
    simple_graph_ext(graph_complement(empty_graph[V]), complete_graph[V])
}

/// The complement of the complete graph is the empty graph.
theorem graph_complement_complete_graph[V] {
    graph_complement(complete_graph[V]) = empty_graph[V]
} by {
    forall(x: V, y: V) {
        graph_complement_adj_eq(complete_graph[V], x, y)
        complete_graph_adj_iff_ne[V](x, y)
        empty_graph_adj_false[V](x, y)
        if graph_complement(complete_graph[V]).adj(x, y) {
            x != y
            not complete_graph[V].adj(x, y)
            complete_graph[V].adj(x, y)
            false
        }
        if empty_graph[V].adj(x, y) {
            false
        }
        graph_complement(complete_graph[V]).adj(x, y) = empty_graph[V].adj(x, y)
    }
    simple_graph_ext(graph_complement(complete_graph[V]), empty_graph[V])
}

/// Complementation is an involution on simple graphs.
theorem graph_complement_involutive[V](g: SimpleGraph[V]) {
    graph_complement(graph_complement(g)) = g
} by {
    forall(x: V, y: V) {
        graph_complement_adj_eq(graph_complement(g), x, y)
        graph_complement_adj_eq(g, x, y)
        if graph_complement(graph_complement(g)).adj(x, y) {
            x != y
            not graph_complement(g).adj(x, y)
            if not g.adj(x, y) {
                graph_complement(g).adj(x, y)
                false
            }
            g.adj(x, y)
        }
        if g.adj(x, y) {
            simple_graph_adj_ne(g, x, y)
            x != y
            if graph_complement(g).adj(x, y) {
                not g.adj(x, y)
                false
            }
            not graph_complement(g).adj(x, y)
            graph_complement(graph_complement(g)).adj(x, y)
        }
        graph_complement(graph_complement(g)).adj(x, y) = g.adj(x, y)
    }
    simple_graph_ext(graph_complement(graph_complement(g)), g)
}

/// The union of a graph with the empty graph is the original graph.
theorem graph_union_empty_graph[V](g: SimpleGraph[V]) {
    graph_union(g, empty_graph[V]) = g
} by {
    forall(x: V, y: V) {
        graph_union_adj_eq(g, empty_graph[V], x, y)
        empty_graph_adj_false[V](x, y)
        graph_union(g, empty_graph[V]).adj(x, y) = g.adj(x, y)
    }
    simple_graph_ext(graph_union(g, empty_graph[V]), g)
}

/// The union of a graph with the complete graph is the complete graph.
theorem graph_union_complete_graph[V](g: SimpleGraph[V]) {
    graph_union(g, complete_graph[V]) = complete_graph[V]
} by {
    forall(x: V, y: V) {
        graph_union_adj_eq(g, complete_graph[V], x, y)
        complete_graph_adj_iff_ne[V](x, y)
        if graph_union(g, complete_graph[V]).adj(x, y) {
            if g.adj(x, y) {
                simple_graph_adj_ne(g, x, y)
                x != y
                complete_graph[V].adj(x, y)
            } else {
                complete_graph[V].adj(x, y)
            }
        }
        if complete_graph[V].adj(x, y) {
            graph_union(g, complete_graph[V]).adj(x, y)
        }
        graph_union(g, complete_graph[V]).adj(x, y) = complete_graph[V].adj(x, y)
    }
    simple_graph_ext(graph_union(g, complete_graph[V]), complete_graph[V])
}

/// The intersection of a graph with the empty graph is the empty graph.
theorem graph_intersection_empty_graph[V](g: SimpleGraph[V]) {
    graph_intersection(g, empty_graph[V]) = empty_graph[V]
} by {
    forall(x: V, y: V) {
        graph_intersection_adj_eq(g, empty_graph[V], x, y)
        empty_graph_adj_false[V](x, y)
        graph_intersection(g, empty_graph[V]).adj(x, y) = empty_graph[V].adj(x, y)
    }
    simple_graph_ext(graph_intersection(g, empty_graph[V]), empty_graph[V])
}

/// The intersection of a graph with the complete graph is the original graph.
theorem graph_intersection_complete_graph[V](g: SimpleGraph[V]) {
    graph_intersection(g, complete_graph[V]) = g
} by {
    forall(x: V, y: V) {
        graph_intersection_adj_eq(g, complete_graph[V], x, y)
        complete_graph_adj_iff_ne[V](x, y)
        if g.adj(x, y) {
            simple_graph_adj_ne(g, x, y)
            x != y
            complete_graph[V].adj(x, y)
            graph_intersection(g, complete_graph[V]).adj(x, y)
        }
        if graph_intersection(g, complete_graph[V]).adj(x, y) {
            g.adj(x, y)
        }
        graph_intersection(g, complete_graph[V]).adj(x, y) = g.adj(x, y)
    }
    simple_graph_ext(graph_intersection(g, complete_graph[V]), g)
}

/// Union of simple graphs on the same vertices is commutative.
theorem graph_union_comm[V](g: SimpleGraph[V], h: SimpleGraph[V]) {
    graph_union(g, h) = graph_union(h, g)
} by {
    forall(x: V, y: V) {
        graph_union_adj_eq(g, h, x, y)
        graph_union_adj_eq(h, g, x, y)
        if graph_union(g, h).adj(x, y) {
            if g.adj(x, y) {
                graph_union(h, g).adj(x, y)
            } else {
                h.adj(x, y)
                graph_union(h, g).adj(x, y)
            }
        }
        if graph_union(h, g).adj(x, y) {
            if h.adj(x, y) {
                graph_union(g, h).adj(x, y)
            } else {
                g.adj(x, y)
                graph_union(g, h).adj(x, y)
            }
        }
        graph_union(g, h).adj(x, y) = graph_union(h, g).adj(x, y)
    }
    simple_graph_ext(graph_union(g, h), graph_union(h, g))
}

/// Union of a simple graph with itself is the original graph.
theorem graph_union_idempotent[V](g: SimpleGraph[V]) {
    graph_union(g, g) = g
} by {
    forall(x: V, y: V) {
        graph_union_adj_eq(g, g, x, y)
        if graph_union(g, g).adj(x, y) {
            g.adj(x, y)
        }
        if g.adj(x, y) {
            graph_union(g, g).adj(x, y)
        }
        graph_union(g, g).adj(x, y) = g.adj(x, y)
    }
    simple_graph_ext(graph_union(g, g), g)
}

/// Intersection of simple graphs on the same vertices is commutative.
theorem graph_intersection_comm[V](g: SimpleGraph[V], h: SimpleGraph[V]) {
    graph_intersection(g, h) = graph_intersection(h, g)
} by {
    forall(x: V, y: V) {
        graph_intersection_adj_eq(g, h, x, y)
        graph_intersection_adj_eq(h, g, x, y)
        if graph_intersection(g, h).adj(x, y) {
            g.adj(x, y)
            h.adj(x, y)
            graph_intersection(h, g).adj(x, y)
        }
        if graph_intersection(h, g).adj(x, y) {
            h.adj(x, y)
            g.adj(x, y)
            graph_intersection(g, h).adj(x, y)
        }
        graph_intersection(g, h).adj(x, y) = graph_intersection(h, g).adj(x, y)
    }
    simple_graph_ext(graph_intersection(g, h), graph_intersection(h, g))
}

/// Intersection of a simple graph with itself is the original graph.
theorem graph_intersection_idempotent[V](g: SimpleGraph[V]) {
    graph_intersection(g, g) = g
} by {
    forall(x: V, y: V) {
        graph_intersection_adj_eq(g, g, x, y)
        if graph_intersection(g, g).adj(x, y) {
            g.adj(x, y)
        }
        if g.adj(x, y) {
            graph_intersection(g, g).adj(x, y)
        }
        graph_intersection(g, g).adj(x, y) = g.adj(x, y)
    }
    simple_graph_ext(graph_intersection(g, g), g)
}

/// A vertex set is a clique when every two distinct elements are adjacent.
define is_clique[V](g: SimpleGraph[V], s: Set[V]) -> Bool {
    forall(x: V, y: V) {
        s.contains(x) and s.contains(y) and x != y implies g.adj(x, y)
    }
}

/// A vertex set is independent when no two distinct elements are adjacent.
define is_independent_set[V](g: SimpleGraph[V], s: Set[V]) -> Bool {
    forall(x: V, y: V) {
        s.contains(x) and s.contains(y) and x != y implies not g.adj(x, y)
    }
}

/// Every subset of a clique is a clique.
theorem is_clique_subset[V](g: SimpleGraph[V], s: Set[V], t: Set[V]) {
    is_clique(g, t) and s.subset(t) implies is_clique(g, s)
} by {
    if is_clique(g, t) and s.subset(t) {
        is_clique(g, t) = forall(x: V, y: V) {
            t.contains(x) and t.contains(y) and x != y implies g.adj(x, y)
        }
        forall(x: V, y: V) {
            if s.contains(x) and s.contains(y) and x != y {
                subset_contains(s, t, x)
                subset_contains(s, t, y)
                t.contains(x)
                t.contains(y)
                t.contains(x) and t.contains(y) and x != y
                g.adj(x, y)
            }
        }
        is_clique(g, s)
    }
}

/// Every subset of an independent set is independent.
theorem is_independent_set_subset[V](g: SimpleGraph[V], s: Set[V], t: Set[V]) {
    is_independent_set(g, t) and s.subset(t) implies is_independent_set(g, s)
} by {
    if is_independent_set(g, t) and s.subset(t) {
        is_independent_set(g, t) = forall(x: V, y: V) {
            t.contains(x) and t.contains(y) and x != y implies not g.adj(x, y)
        }
        forall(x: V, y: V) {
            if s.contains(x) and s.contains(y) and x != y {
                subset_contains(s, t, x)
                subset_contains(s, t, y)
                t.contains(x)
                t.contains(y)
                t.contains(x) and t.contains(y) and x != y
                not g.adj(x, y)
            }
        }
        is_independent_set(g, s)
    }
}

/// Two distinct members of a clique are adjacent.
theorem is_clique_contains_adj[V](g: SimpleGraph[V], s: Set[V], x: V, y: V) {
    is_clique(g, s) and s.contains(x) and s.contains(y) and x != y implies g.adj(x, y)
} by {
    if is_clique(g, s) and s.contains(x) and s.contains(y) and x != y {
        is_clique(g, s) = forall(a: V, b: V) {
            s.contains(a) and s.contains(b) and a != b implies g.adj(a, b)
        }
        g.adj(x, y)
    }
}

/// Two distinct members of an independent set are not adjacent.
theorem is_independent_set_contains_not_adj[V](g: SimpleGraph[V], s: Set[V], x: V, y: V) {
    is_independent_set(g, s) and s.contains(x) and s.contains(y) and x != y implies not g.adj(x, y)
} by {
    if is_independent_set(g, s) and s.contains(x) and s.contains(y) and x != y {
        is_independent_set(g, s) = forall(a: V, b: V) {
            s.contains(a) and s.contains(b) and a != b implies not g.adj(a, b)
        }
        not g.adj(x, y)
    }
}

/// Every vertex set is a clique in the complete graph.
theorem complete_graph_is_clique[V](s: Set[V]) {
    is_clique(complete_graph[V], s)
} by {
    forall(x: V, y: V) {
        if s.contains(x) and s.contains(y) and x != y {
            complete_graph_adj_iff_ne[V](x, y)
            complete_graph[V].adj(x, y)
        }
    }
}

/// Every vertex set is independent in the empty graph.
theorem empty_graph_is_independent_set[V](s: Set[V]) {
    is_independent_set(empty_graph[V], s)
} by {
    forall(x: V, y: V) {
        if s.contains(x) and s.contains(y) and x != y {
            empty_graph_adj_false[V](x, y)
        }
    }
}

/// Cliques in the complement are exactly independent sets in the original graph.
theorem is_clique_graph_complement_eq_independent_set[V](g: SimpleGraph[V], s: Set[V]) {
    is_clique(graph_complement(g), s) = is_independent_set(g, s)
} by {
    if is_clique(graph_complement(g), s) {
        is_clique(graph_complement(g), s) = forall(x: V, y: V) {
            s.contains(x) and s.contains(y) and x != y implies graph_complement(g).adj(x, y)
        }
        forall(x: V, y: V) {
            if s.contains(x) and s.contains(y) and x != y {
                graph_complement(g).adj(x, y)
                graph_complement_adj_eq(g, x, y)
                not g.adj(x, y)
            }
        }
        is_independent_set(g, s)
    }
    if is_independent_set(g, s) {
        is_independent_set(g, s) = forall(x: V, y: V) {
            s.contains(x) and s.contains(y) and x != y implies not g.adj(x, y)
        }
        forall(x: V, y: V) {
            if s.contains(x) and s.contains(y) and x != y {
                is_independent_set_contains_not_adj(g, s, x, y)
                not g.adj(x, y)
                graph_complement_adj_eq(g, x, y)
                graph_complement(g).adj(x, y)
            }
        }
        is_clique(graph_complement(g), s)
    }
}

/// Independent sets in the complement are exactly cliques in the original graph.
theorem is_independent_set_graph_complement_eq_clique[V](g: SimpleGraph[V], s: Set[V]) {
    is_independent_set(graph_complement(g), s) = is_clique(g, s)
} by {
    if is_independent_set(graph_complement(g), s) {
        is_independent_set(graph_complement(g), s) = forall(x: V, y: V) {
            s.contains(x) and s.contains(y) and x != y implies not graph_complement(g).adj(x, y)
        }
        forall(x: V, y: V) {
            if s.contains(x) and s.contains(y) and x != y {
                not graph_complement(g).adj(x, y)
                graph_complement_adj_eq(g, x, y)
                if not g.adj(x, y) {
                    graph_complement(g).adj(x, y)
                    false
                }
                g.adj(x, y)
            }
        }
        is_clique(g, s) = forall(x: V, y: V) {
            s.contains(x) and s.contains(y) and x != y implies g.adj(x, y)
        }
        is_clique(g, s)
    }
    if is_clique(g, s) {
        is_clique(g, s) = forall(x: V, y: V) {
            s.contains(x) and s.contains(y) and x != y implies g.adj(x, y)
        }
        forall(x: V, y: V) {
            if s.contains(x) and s.contains(y) and x != y {
                is_clique_contains_adj(g, s, x, y)
                g.adj(x, y)
                graph_complement_adj_eq(g, x, y)
                not graph_complement(g).adj(x, y)
            }
        }
        is_independent_set(graph_complement(g), s) = forall(x: V, y: V) {
            s.contains(x) and s.contains(y) and x != y implies not graph_complement(g).adj(x, y)
        }
        is_independent_set(graph_complement(g), s)
    }
}

/// The adjacency relation of the subgraph induced by a vertex set.
define induced_subgraph_adj[V](g: SimpleGraph[V], s: Set[V], x: V, y: V) -> Bool {
    g.adj(x, y) and s.contains(x) and s.contains(y)
}

/// The induced-subgraph adjacency relation is symmetric.
theorem induced_subgraph_adj_is_symmetric[V](g: SimpleGraph[V], s: Set[V]) {
    is_symmetric(induced_subgraph_adj(g, s))
} by {
    forall(x: V, y: V) {
        if induced_subgraph_adj(g, s, x, y) {
            induced_subgraph_adj(g, s, x, y) =
                (g.adj(x, y) and s.contains(x) and s.contains(y))
            g.adj(x, y)
            s.contains(x)
            s.contains(y)
            simple_graph_adj_symmetric(g, x, y)
            g.adj(y, x)
            g.adj(y, x) and s.contains(y) and s.contains(x)
            induced_subgraph_adj(g, s, y, x)
        }
    }
}

/// The induced-subgraph adjacency relation has no loops.
theorem induced_subgraph_adj_is_irreflexive[V](g: SimpleGraph[V], s: Set[V]) {
    is_irreflexive(induced_subgraph_adj(g, s))
} by {
    forall(x: V) {
        if induced_subgraph_adj(g, s, x, x) {
            induced_subgraph_adj(g, s, x, x) =
                (g.adj(x, x) and s.contains(x) and s.contains(x))
            g.adj(x, x)
            simple_graph_adj_irreflexive(g, x)
            false
        }
    }
}

/// The induced-subgraph adjacency relation satisfies the simple-graph constraint.
theorem induced_subgraph_adj_constraint[V](g: SimpleGraph[V], s: Set[V]) {
    is_symmetric(induced_subgraph_adj(g, s)) and is_irreflexive(induced_subgraph_adj(g, s))
} by {
    induced_subgraph_adj_is_symmetric(g, s)
    induced_subgraph_adj_is_irreflexive(g, s)
}

/// The subgraph of a simple graph induced by a vertex set.
let induced_subgraph[V](g: SimpleGraph[V], s: Set[V]) -> result: SimpleGraph[V] satisfy {
    SimpleGraph.new(induced_subgraph_adj(g, s)) = Option.some(result)
}

/// Adjacency in an induced subgraph is adjacency with both endpoints in the vertex set.
theorem induced_subgraph_adj_eq[V](g: SimpleGraph[V], s: Set[V], x: V, y: V) {
    induced_subgraph(g, s).adj(x, y) = (g.adj(x, y) and s.contains(x) and s.contains(y))
} by {
    induced_subgraph(g, s).adj = induced_subgraph_adj(g, s)
    induced_subgraph(g, s).adj(x, y) = induced_subgraph_adj(g, s, x, y)
    induced_subgraph_adj(g, s, x, y) = (g.adj(x, y) and s.contains(x) and s.contains(y))
}

/// Adjacent vertices in an induced subgraph are adjacent in the original graph.
theorem induced_subgraph_adj_imp_base_adj[V](g: SimpleGraph[V], s: Set[V], x: V, y: V) {
    induced_subgraph(g, s).adj(x, y) implies g.adj(x, y)
} by {
    if induced_subgraph(g, s).adj(x, y) {
        induced_subgraph_adj_eq(g, s, x, y)
        g.adj(x, y)
    }
}

/// The left endpoint of an induced-subgraph edge lies in the inducing vertex set.
theorem induced_subgraph_adj_imp_left_mem[V](g: SimpleGraph[V], s: Set[V], x: V, y: V) {
    induced_subgraph(g, s).adj(x, y) implies s.contains(x)
} by {
    if induced_subgraph(g, s).adj(x, y) {
        induced_subgraph_adj_eq(g, s, x, y)
        s.contains(x)
    }
}

/// The right endpoint of an induced-subgraph edge lies in the inducing vertex set.
theorem induced_subgraph_adj_imp_right_mem[V](g: SimpleGraph[V], s: Set[V], x: V, y: V) {
    induced_subgraph(g, s).adj(x, y) implies s.contains(y)
} by {
    if induced_subgraph(g, s).adj(x, y) {
        induced_subgraph_adj_eq(g, s, x, y)
        s.contains(y)
    }
}

/// An original edge whose endpoints lie in the vertex set is an induced-subgraph edge.
theorem induced_subgraph_adj_of_base_adj[V](g: SimpleGraph[V], s: Set[V], x: V, y: V) {
    g.adj(x, y) and s.contains(x) and s.contains(y) implies induced_subgraph(g, s).adj(x, y)
} by {
    if g.adj(x, y) and s.contains(x) and s.contains(y) {
        induced_subgraph_adj_eq(g, s, x, y)
        induced_subgraph(g, s).adj(x, y)
    }
}

/// On a set contained in the inducing set, cliquehood is unchanged by taking the induced subgraph.
theorem is_clique_induced_subgraph_eq[V](g: SimpleGraph[V], s: Set[V], t: Set[V]) {
    t.subset(s) implies is_clique(induced_subgraph(g, s), t) = is_clique(g, t)
} by {
    if t.subset(s) {
        if is_clique(induced_subgraph(g, s), t) {
            is_clique(induced_subgraph(g, s), t) = forall(x: V, y: V) {
                t.contains(x) and t.contains(y) and x != y implies induced_subgraph(g, s).adj(x, y)
            }
            forall(x: V, y: V) {
                if t.contains(x) and t.contains(y) and x != y {
                    induced_subgraph(g, s).adj(x, y)
                    induced_subgraph_adj_imp_base_adj(g, s, x, y)
                    g.adj(x, y)
                }
            }
            is_clique(g, t)
        }
        if is_clique(g, t) {
            is_clique(g, t) = forall(x: V, y: V) {
                t.contains(x) and t.contains(y) and x != y implies g.adj(x, y)
            }
            forall(x: V, y: V) {
                if t.contains(x) and t.contains(y) and x != y {
                    is_clique_contains_adj(g, t, x, y)
                    g.adj(x, y)
                    subset_contains(t, s, x)
                    subset_contains(t, s, y)
                    s.contains(x)
                    s.contains(y)
                    induced_subgraph_adj_of_base_adj(g, s, x, y)
                    induced_subgraph(g, s).adj(x, y)
                }
            }
            is_clique(induced_subgraph(g, s), t)
        }
        is_clique(induced_subgraph(g, s), t) = is_clique(g, t)
    }
}

/// On a set contained in the inducing set, independence is unchanged by taking the induced subgraph.
theorem is_independent_set_induced_subgraph_eq[V](g: SimpleGraph[V], s: Set[V], t: Set[V]) {
    t.subset(s) implies is_independent_set(induced_subgraph(g, s), t) = is_independent_set(g, t)
} by {
    if t.subset(s) {
        if is_independent_set(induced_subgraph(g, s), t) {
            is_independent_set(induced_subgraph(g, s), t) = forall(x: V, y: V) {
                t.contains(x) and t.contains(y) and x != y implies not induced_subgraph(g, s).adj(x, y)
            }
            forall(x: V, y: V) {
                if t.contains(x) and t.contains(y) and x != y {
                    not induced_subgraph(g, s).adj(x, y)
                    subset_contains(t, s, x)
                    subset_contains(t, s, y)
                    s.contains(x)
                    s.contains(y)
                    if g.adj(x, y) {
                        induced_subgraph_adj_of_base_adj(g, s, x, y)
                        induced_subgraph(g, s).adj(x, y)
                        false
                    }
                    not g.adj(x, y)
                }
            }
            is_independent_set(g, t)
        }
        if is_independent_set(g, t) {
            is_independent_set(g, t) = forall(x: V, y: V) {
                t.contains(x) and t.contains(y) and x != y implies not g.adj(x, y)
            }
            forall(x: V, y: V) {
                if t.contains(x) and t.contains(y) and x != y {
                    is_independent_set_contains_not_adj(g, t, x, y)
                    not g.adj(x, y)
                    if induced_subgraph(g, s).adj(x, y) {
                        induced_subgraph_adj_imp_base_adj(g, s, x, y)
                        g.adj(x, y)
                        false
                    }
                    not induced_subgraph(g, s).adj(x, y)
                }
            }
            is_independent_set(induced_subgraph(g, s), t)
        }
        is_independent_set(induced_subgraph(g, s), t) = is_independent_set(g, t)
    }
}

/// True if a vertex map preserves adjacency.
define is_graph_hom[V, W](g: SimpleGraph[V], h: SimpleGraph[W], f: V -> W) -> Bool {
    forall(x: V, y: V) {
        g.adj(x, y) implies h.adj(f(x), f(y))
    }
}

/// A homomorphism of simple graphs.
structure SimpleGraphHom[V, W] {
    /// The source graph.
    src: SimpleGraph[V]

    /// The target graph.
    dst: SimpleGraph[W]

    /// The map on vertices.
    map: V -> W
} constraint {
    is_graph_hom(src, dst, map)
}

/// A graph homomorphism preserves adjacency.
theorem simple_graph_hom_maps_adj[V, W](f: SimpleGraphHom[V, W], x: V, y: V) {
    f.src.adj(x, y) implies f.dst.adj(f.map(x), f.map(y))
} by {
    if f.src.adj(x, y) {
        SimpleGraphHom.constraint(f.src, f.dst, f.map)
        is_graph_hom(f.src, f.dst, f.map) =
            forall(a: V, b: V) {
                f.src.adj(a, b) implies f.dst.adj(f.map(a), f.map(b))
            }
        f.dst.adj(f.map(x), f.map(y))
    }
}

/// The image of a vertex set under the vertex map of a graph homomorphism.
define simple_graph_hom_image_set[V, W](f: SimpleGraphHom[V, W], s: Set[V]) -> Set[W] {
    set_image(s, f.map)
}

/// The preimage of a vertex set under the vertex map of a graph homomorphism.
define simple_graph_hom_preimage_set[V, W](f: SimpleGraphHom[V, W], t: Set[W]) -> Set[V] {
    set_preimage(f.map, t)
}

/// The image graph of an induced source subgraph under a graph homomorphism.
define simple_graph_hom_image[V, W](f: SimpleGraphHom[V, W], s: Set[V]) -> SimpleGraph[W] {
    induced_subgraph(f.dst, simple_graph_hom_image_set(f, s))
}

/// The preimage graph of an induced target subgraph under a graph homomorphism.
define simple_graph_hom_preimage[V, W](f: SimpleGraphHom[V, W], t: Set[W]) -> SimpleGraph[V] {
    induced_subgraph(f.src, simple_graph_hom_preimage_set(f, t))
}

/// The image set of a graph homomorphism is the direct image of the vertex set.
theorem simple_graph_hom_image_set_eq[V, W](f: SimpleGraphHom[V, W], s: Set[V]) {
    simple_graph_hom_image_set(f, s) = set_image(s, f.map)
}

/// The preimage set of a graph homomorphism is the inverse image of the vertex set.
theorem simple_graph_hom_preimage_set_eq[V, W](f: SimpleGraphHom[V, W], t: Set[W]) {
    simple_graph_hom_preimage_set(f, t) = set_preimage(f.map, t)
}

/// A vertex in the source set maps into the image set.
theorem simple_graph_hom_image_set_contains_map[V, W](f: SimpleGraphHom[V, W], s: Set[V], x: V) {
    s.contains(x) implies simple_graph_hom_image_set(f, s).contains(f.map(x))
} by {
    if s.contains(x) {
        simple_graph_hom_image_set_eq(f, s)
        maps_into_set_image(s, f.map, x)
        set_image(s, f.map).contains(f.map(x))
        simple_graph_hom_image_set(f, s).contains(f.map(x))
    }
}

/// A vertex lies in a homomorphic preimage set exactly when its image lies in the target set.
theorem simple_graph_hom_preimage_set_contains_eq[V, W](f: SimpleGraphHom[V, W], t: Set[W], x: V) {
    simple_graph_hom_preimage_set(f, t).contains(x) = t.contains(f.map(x))
} by {
    simple_graph_hom_preimage_set_eq(f, t)
    set_preimage_contains_eq(f.map, t, x)
    simple_graph_hom_preimage_set(f, t).contains(x) = set_preimage(f.map, t).contains(x)
    set_preimage(f.map, t).contains(x) = t.contains(f.map(x))
}

/// The image graph is the target graph induced by the image vertex set.
theorem simple_graph_hom_image_eq[V, W](f: SimpleGraphHom[V, W], s: Set[V]) {
    simple_graph_hom_image(f, s) = induced_subgraph(f.dst, simple_graph_hom_image_set(f, s))
}

/// The preimage graph is the source graph induced by the preimage vertex set.
theorem simple_graph_hom_preimage_eq[V, W](f: SimpleGraphHom[V, W], t: Set[W]) {
    simple_graph_hom_preimage(f, t) = induced_subgraph(f.src, simple_graph_hom_preimage_set(f, t))
}

/// Adjacency in the image graph is target adjacency with both endpoints in the image set.
theorem simple_graph_hom_image_adj_eq[V, W](f: SimpleGraphHom[V, W], s: Set[V], x: W, y: W) {
    simple_graph_hom_image(f, s).adj(x, y) =
        (f.dst.adj(x, y) and simple_graph_hom_image_set(f, s).contains(x)
            and simple_graph_hom_image_set(f, s).contains(y))
} by {
    simple_graph_hom_image_eq(f, s)
    induced_subgraph_adj_eq(f.dst, simple_graph_hom_image_set(f, s), x, y)
}

/// Adjacency in the preimage graph is source adjacency with both images in the target set.
theorem simple_graph_hom_preimage_adj_eq[V, W](f: SimpleGraphHom[V, W], t: Set[W], x: V, y: V) {
    simple_graph_hom_preimage(f, t).adj(x, y) =
        (f.src.adj(x, y) and t.contains(f.map(x)) and t.contains(f.map(y)))
} by {
    simple_graph_hom_preimage_eq(f, t)
    induced_subgraph_adj_eq(f.src, simple_graph_hom_preimage_set(f, t), x, y)
    simple_graph_hom_preimage_set_contains_eq(f, t, x)
    simple_graph_hom_preimage_set_contains_eq(f, t, y)
}

/// The homomorphism maps the induced source subgraph into its image graph.
theorem simple_graph_hom_maps_induced_to_image[V, W](f: SimpleGraphHom[V, W], s: Set[V]) {
    is_graph_hom(induced_subgraph(f.src, s), simple_graph_hom_image(f, s), f.map)
} by {
    forall(x: V, y: V) {
        if induced_subgraph(f.src, s).adj(x, y) {
            induced_subgraph_adj_imp_base_adj(f.src, s, x, y)
            f.src.adj(x, y)
            simple_graph_hom_maps_adj(f, x, y)
            f.dst.adj(f.map(x), f.map(y))
            induced_subgraph_adj_imp_left_mem(f.src, s, x, y)
            s.contains(x)
            induced_subgraph_adj_imp_right_mem(f.src, s, x, y)
            s.contains(y)
            simple_graph_hom_image_set_contains_map(f, s, x)
            simple_graph_hom_image_set(f, s).contains(f.map(x))
            simple_graph_hom_image_set_contains_map(f, s, y)
            simple_graph_hom_image_set(f, s).contains(f.map(y))
            simple_graph_hom_image_adj_eq(f, s, f.map(x), f.map(y))
            simple_graph_hom_image(f, s).adj(f.map(x), f.map(y))
        }
    }
}

/// The homomorphism maps the preimage graph into the induced target graph.
theorem simple_graph_hom_maps_preimage_to_induced[V, W](f: SimpleGraphHom[V, W], t: Set[W]) {
    is_graph_hom(simple_graph_hom_preimage(f, t), induced_subgraph(f.dst, t), f.map)
} by {
    forall(x: V, y: V) {
        if simple_graph_hom_preimage(f, t).adj(x, y) {
            simple_graph_hom_preimage_adj_eq(f, t, x, y)
            f.src.adj(x, y)
            t.contains(f.map(x))
            t.contains(f.map(y))
            simple_graph_hom_maps_adj(f, x, y)
            f.dst.adj(f.map(x), f.map(y))
            induced_subgraph_adj_of_base_adj(f.dst, t, f.map(x), f.map(y))
            induced_subgraph(f.dst, t).adj(f.map(x), f.map(y))
        }
    }
}

/// The identity map is a graph homomorphism.
theorem identity_fn_is_graph_hom[V](g: SimpleGraph[V]) {
    is_graph_hom(g, g, identity_fn[V])
} by {
    forall(x: V, y: V) {
        if g.adj(x, y) {
            identity_fn[V](x) = x
            identity_fn[V](y) = y
            g.adj(identity_fn[V](x), identity_fn[V](y))
        }
    }
}

/// Composition of graph homomorphism predicates.
theorem is_graph_hom_compose[U, V, W](
    g: SimpleGraph[U],
    h: SimpleGraph[V],
    k: SimpleGraph[W],
    f: V -> W,
    e: U -> V
) {
    is_graph_hom(h, k, f) and is_graph_hom(g, h, e) implies
        is_graph_hom(g, k, compose(f, e))
} by {
    if is_graph_hom(h, k, f) and is_graph_hom(g, h, e) {
        is_graph_hom(h, k, f) =
            forall(a: V, b: V) {
                h.adj(a, b) implies k.adj(f(a), f(b))
            }
        is_graph_hom(g, h, e) =
            forall(a: U, b: U) {
                g.adj(a, b) implies h.adj(e(a), e(b))
            }
        forall(x: U, y: U) {
            if g.adj(x, y) {
                h.adj(e(x), e(y))
                k.adj(f(e(x)), f(e(y)))
                compose(f, e, x) = f(e(x))
                compose(f, e, y) = f(e(y))
                k.adj(compose(f, e, x), compose(f, e, y))
            }
        }
    }
}

/// True if a vertex map reflects adjacency.
define reflects_graph_adj[V, W](g: SimpleGraph[V], h: SimpleGraph[W], f: V -> W) -> Bool {
    forall(x: V, y: V) {
        h.adj(f(x), f(y)) implies g.adj(x, y)
    }
}

/// An adjacency-reflecting vertex map carries target adjacency back to source adjacency.
theorem reflects_graph_adj_apply[V, W](
    g: SimpleGraph[V], h: SimpleGraph[W], f: V -> W, x: V, y: V
) {
    reflects_graph_adj(g, h, f) and h.adj(f(x), f(y)) implies g.adj(x, y)
} by {
    if reflects_graph_adj(g, h, f) and h.adj(f(x), f(y)) {
        reflects_graph_adj(g, h, f) = forall(a: V, b: V) {
            h.adj(f(a), f(b)) implies g.adj(a, b)
        }
        h.adj(f(x), f(y))
        h.adj(f(x), f(y)) implies g.adj(x, y)
        g.adj(x, y)
    }
}

/// True if a vertex map is an injective adjacency-preserving and adjacency-reflecting map.
define is_graph_embedding[V, W](g: SimpleGraph[V], h: SimpleGraph[W], f: V -> W) -> Bool {
    is_graph_hom(g, h, f) and reflects_graph_adj(g, h, f) and is_injective_fn(f)
}

/// A graph embedding preserves adjacency.
theorem graph_embedding_is_hom[V, W](g: SimpleGraph[V], h: SimpleGraph[W], f: V -> W) {
    is_graph_embedding(g, h, f) implies is_graph_hom(g, h, f)
} by {
    if is_graph_embedding(g, h, f) {
        is_graph_embedding(g, h, f) =
            (is_graph_hom(g, h, f) and reflects_graph_adj(g, h, f) and is_injective_fn(f))
        is_graph_hom(g, h, f)
    }
}

/// A graph embedding reflects adjacency.
theorem graph_embedding_reflects_adj[V, W](g: SimpleGraph[V], h: SimpleGraph[W], f: V -> W) {
    is_graph_embedding(g, h, f) implies reflects_graph_adj(g, h, f)
} by {
    if is_graph_embedding(g, h, f) {
        is_graph_embedding(g, h, f) =
            (is_graph_hom(g, h, f) and reflects_graph_adj(g, h, f) and is_injective_fn(f))
        reflects_graph_adj(g, h, f)
    }
}

/// A graph embedding is injective on vertices.
theorem graph_embedding_is_injective[V, W](g: SimpleGraph[V], h: SimpleGraph[W], f: V -> W) {
    is_graph_embedding(g, h, f) implies is_injective_fn(f)
} by {
    if is_graph_embedding(g, h, f) {
        is_graph_embedding(g, h, f) =
            (is_graph_hom(g, h, f) and reflects_graph_adj(g, h, f) and is_injective_fn(f))
        is_injective_fn(f)
    }
}

/// True if two vertex maps are inverse on both graph vertex types.
define are_inverse_vertex_maps[V, W](f: V -> W, e: W -> V) -> Bool {
    (forall(x: V) { e(f(x)) = x }) and (forall(y: W) { f(e(y)) = y })
}

/// True if a graph homomorphism has a graph homomorphism inverse.
define is_graph_iso_pair[V, W](
    g: SimpleGraph[V],
    h: SimpleGraph[W],
    f: V -> W,
    e: W -> V
) -> Bool {
    is_graph_hom(g, h, f) and is_graph_hom(h, g, e) and are_inverse_vertex_maps(f, e)
}

/// A graph embedding: an injective vertex map preserving and reflecting adjacency.
structure SimpleGraphEmbedding[V, W] {
    /// The source graph.
    src: SimpleGraph[V]

    /// The target graph.
    dst: SimpleGraph[W]

    /// The map on vertices.
    map: V -> W
} constraint {
    is_graph_embedding(src, dst, map)
}

/// A graph embedding preserves adjacency.
theorem simple_graph_embedding_is_hom[V, W](f: SimpleGraphEmbedding[V, W]) {
    is_graph_hom(f.src, f.dst, f.map)
} by {
    SimpleGraphEmbedding.constraint(f.src, f.dst, f.map)
    is_graph_embedding(f.src, f.dst, f.map) =
        (is_graph_hom(f.src, f.dst, f.map) and reflects_graph_adj(f.src, f.dst, f.map) and is_injective_fn(f.map))
}

/// A graph embedding reflects adjacency.
theorem simple_graph_embedding_reflects_adj[V, W](f: SimpleGraphEmbedding[V, W]) {
    reflects_graph_adj(f.src, f.dst, f.map)
} by {
    SimpleGraphEmbedding.constraint(f.src, f.dst, f.map)
    is_graph_embedding(f.src, f.dst, f.map) =
        (is_graph_hom(f.src, f.dst, f.map) and reflects_graph_adj(f.src, f.dst, f.map) and is_injective_fn(f.map))
}

/// A graph embedding is injective on vertices.
theorem simple_graph_embedding_is_injective[V, W](f: SimpleGraphEmbedding[V, W]) {
    is_injective_fn(f.map)
} by {
    SimpleGraphEmbedding.constraint(f.src, f.dst, f.map)
    is_graph_embedding(f.src, f.dst, f.map) =
        (is_graph_hom(f.src, f.dst, f.map) and reflects_graph_adj(f.src, f.dst, f.map) and is_injective_fn(f.map))
}

/// A graph embedding maps adjacent vertices to adjacent vertices.
theorem simple_graph_embedding_maps_adj[V, W](f: SimpleGraphEmbedding[V, W], x: V, y: V) {
    f.src.adj(x, y) implies f.dst.adj(f.map(x), f.map(y))
} by {
    simple_graph_embedding_is_hom(f)
    is_graph_hom(f.src, f.dst, f.map) =
        forall(a: V, b: V) {
            f.src.adj(a, b) implies f.dst.adj(f.map(a), f.map(b))
        }
}

/// A graph embedding reflects adjacency on vertices.
theorem simple_graph_embedding_map_adj_iff[V, W](f: SimpleGraphEmbedding[V, W], x: V, y: V) {
    f.dst.adj(f.map(x), f.map(y)) implies f.src.adj(x, y)
} by {
    simple_graph_embedding_reflects_adj(f)
    reflects_graph_adj(f.src, f.dst, f.map) =
        forall(a: V, b: V) {
            f.dst.adj(f.map(a), f.map(b)) implies f.src.adj(a, b)
        }
}

/// A graph isomorphism: a pair of mutually inverse adjacency-preserving vertex maps.
structure SimpleGraphIso[V, W] {
    /// The source graph.
    src: SimpleGraph[V]

    /// The target graph.
    dst: SimpleGraph[W]

    /// The forward map on vertices.
    map: V -> W

    /// The inverse map on vertices.
    inv: W -> V
} constraint {
    is_graph_iso_pair(src, dst, map, inv)
}

/// The forward map of a graph isomorphism is a graph homomorphism.
theorem simple_graph_iso_map_is_hom[V, W](f: SimpleGraphIso[V, W]) {
    is_graph_hom(f.src, f.dst, f.map)
} by {
    SimpleGraphIso.constraint(f.src, f.dst, f.map, f.inv)
    is_graph_iso_pair(f.src, f.dst, f.map, f.inv) =
        (is_graph_hom(f.src, f.dst, f.map) and is_graph_hom(f.dst, f.src, f.inv)
            and are_inverse_vertex_maps(f.map, f.inv))
}

/// The inverse map of a graph isomorphism is a graph homomorphism.
theorem simple_graph_iso_inv_is_hom[V, W](f: SimpleGraphIso[V, W]) {
    is_graph_hom(f.dst, f.src, f.inv)
} by {
    SimpleGraphIso.constraint(f.src, f.dst, f.map, f.inv)
    is_graph_iso_pair(f.src, f.dst, f.map, f.inv) =
        (is_graph_hom(f.src, f.dst, f.map) and is_graph_hom(f.dst, f.src, f.inv)
            and are_inverse_vertex_maps(f.map, f.inv))
}

/// The maps of a graph isomorphism are mutually inverse.
theorem simple_graph_iso_inverse_maps[V, W](f: SimpleGraphIso[V, W]) {
    are_inverse_vertex_maps(f.map, f.inv)
} by {
    SimpleGraphIso.constraint(f.src, f.dst, f.map, f.inv)
    is_graph_iso_pair(f.src, f.dst, f.map, f.inv) =
        (is_graph_hom(f.src, f.dst, f.map) and is_graph_hom(f.dst, f.src, f.inv)
            and are_inverse_vertex_maps(f.map, f.inv))
}

/// The inverse map sends `f.map(x)` back to `x`.
theorem simple_graph_iso_left_inv[V, W](f: SimpleGraphIso[V, W], x: V) {
    f.inv(f.map(x)) = x
} by {
    simple_graph_iso_inverse_maps(f)
    are_inverse_vertex_maps(f.map, f.inv) =
        ((forall(a: V) { f.inv(f.map(a)) = a }) and (forall(b: W) { f.map(f.inv(b)) = b }))
}

/// The forward map sends `f.inv(y)` back to `y`.
theorem simple_graph_iso_right_inv[V, W](f: SimpleGraphIso[V, W], y: W) {
    f.map(f.inv(y)) = y
} by {
    simple_graph_iso_inverse_maps(f)
    are_inverse_vertex_maps(f.map, f.inv) =
        ((forall(a: V) { f.inv(f.map(a)) = a }) and (forall(b: W) { f.map(f.inv(b)) = b }))
}

/// Construction of a graph isomorphism remembers the source graph.
theorem simple_graph_iso_new_src[V, W](
    g: SimpleGraph[V],
    h: SimpleGraph[W],
    f: V -> W,
    e: W -> V,
    i: SimpleGraphIso[V, W]
) {
    SimpleGraphIso[V, W].new(g, h, f, e) = Option.some(i) implies i.src = g
}

/// Construction of a graph isomorphism remembers the target graph.
theorem simple_graph_iso_new_dst[V, W](
    g: SimpleGraph[V],
    h: SimpleGraph[W],
    f: V -> W,
    e: W -> V,
    i: SimpleGraphIso[V, W]
) {
    SimpleGraphIso[V, W].new(g, h, f, e) = Option.some(i) implies i.dst = h
}

/// Construction of a graph isomorphism remembers the forward vertex map.
theorem simple_graph_iso_new_map[V, W](
    g: SimpleGraph[V],
    h: SimpleGraph[W],
    f: V -> W,
    e: W -> V,
    i: SimpleGraphIso[V, W]
) {
    SimpleGraphIso[V, W].new(g, h, f, e) = Option.some(i) implies i.map = f
}

/// Construction of a graph isomorphism remembers the inverse vertex map.
theorem simple_graph_iso_new_inv[V, W](
    g: SimpleGraph[V],
    h: SimpleGraph[W],
    f: V -> W,
    e: W -> V,
    i: SimpleGraphIso[V, W]
) {
    SimpleGraphIso[V, W].new(g, h, f, e) = Option.some(i) implies i.inv = e
}

/// The identity vertex map and itself form a graph isomorphism pair.
theorem identity_fn_is_graph_iso_pair[V](g: SimpleGraph[V]) {
    is_graph_iso_pair(g, g, identity_fn[V], identity_fn[V])
} by {
    identity_fn_is_graph_hom(g)
    is_graph_hom(g, g, identity_fn[V])
    forall(x: V) {
        identity_fn[V](identity_fn[V](x)) = x
    }
    are_inverse_vertex_maps(identity_fn[V], identity_fn[V])
    is_graph_iso_pair(g, g, identity_fn[V], identity_fn[V])
}

/// The identity isomorphism of a simple graph.
let simple_graph_iso_refl[V](g: SimpleGraph[V]) -> result: SimpleGraphIso[V, V] satisfy {
    result.src = g and result.dst = g and result.map = identity_fn[V] and result.inv = identity_fn[V]
} by {
    identity_fn_is_graph_iso_pair(g)
    let i: SimpleGraphIso[V, V] satisfy {
        SimpleGraphIso[V, V].new(g, g, identity_fn[V], identity_fn[V]) = Option.some(i)
    }
    simple_graph_iso_new_src(g, g, identity_fn[V], identity_fn[V], i)
    i.src = g
    simple_graph_iso_new_dst(g, g, identity_fn[V], identity_fn[V], i)
    i.dst = g
    simple_graph_iso_new_map(g, g, identity_fn[V], identity_fn[V], i)
    i.map = identity_fn[V]
    simple_graph_iso_new_inv(g, g, identity_fn[V], identity_fn[V], i)
    i.inv = identity_fn[V]
}

/// The identity graph isomorphism has the identity vertex map.
theorem simple_graph_iso_refl_map[V](g: SimpleGraph[V]) {
    simple_graph_iso_refl(g).map = identity_fn[V]
}

/// The identity graph isomorphism has the identity inverse vertex map.
theorem simple_graph_iso_refl_inv[V](g: SimpleGraph[V]) {
    simple_graph_iso_refl(g).inv = identity_fn[V]
}

/// Swapping the maps of a graph isomorphism pair gives another isomorphism pair.
theorem is_graph_iso_pair_swap[V, W](
    g: SimpleGraph[V],
    h: SimpleGraph[W],
    f: V -> W,
    e: W -> V
) {
    is_graph_iso_pair(g, h, f, e) implies is_graph_iso_pair(h, g, e, f)
} by {
    if is_graph_iso_pair(g, h, f, e) {
        is_graph_iso_pair(g, h, f, e) =
            (is_graph_hom(g, h, f) and is_graph_hom(h, g, e)
                and are_inverse_vertex_maps(f, e))
        is_graph_hom(g, h, f) and is_graph_hom(h, g, e) and are_inverse_vertex_maps(f, e)
        is_graph_hom(g, h, f)
        is_graph_hom(h, g, e)
        are_inverse_vertex_maps(f, e)
        are_inverse_vertex_maps(f, e) =
            ((forall(x: V) { e(f(x)) = x }) and (forall(y: W) { f(e(y)) = y }))
        (forall(x: V) { e(f(x)) = x })
        (forall(y: W) { f(e(y)) = y })
        (forall(y: W) { f(e(y)) = y }) and (forall(x: V) { e(f(x)) = x })
        are_inverse_vertex_maps(e, f)
        is_graph_iso_pair(h, g, e, f) =
            (is_graph_hom(h, g, e) and is_graph_hom(g, h, f)
                and are_inverse_vertex_maps(e, f))
        is_graph_iso_pair(h, g, e, f)
    }
}

/// The inverse graph isomorphism.
let simple_graph_iso_symm[V, W](f: SimpleGraphIso[V, W]) -> result: SimpleGraphIso[W, V] satisfy {
    result.src = f.dst and result.dst = f.src and result.map = f.inv and result.inv = f.map
} by {
    SimpleGraphIso.constraint(f.src, f.dst, f.map, f.inv)
    is_graph_iso_pair(f.src, f.dst, f.map, f.inv)
    is_graph_iso_pair_swap(f.src, f.dst, f.map, f.inv)
    is_graph_iso_pair(f.dst, f.src, f.inv, f.map)
    let i: SimpleGraphIso[W, V] satisfy {
        SimpleGraphIso[W, V].new(f.dst, f.src, f.inv, f.map) = Option.some(i)
    }
    simple_graph_iso_new_src(f.dst, f.src, f.inv, f.map, i)
    i.src = f.dst
    simple_graph_iso_new_dst(f.dst, f.src, f.inv, f.map, i)
    i.dst = f.src
    simple_graph_iso_new_map(f.dst, f.src, f.inv, f.map, i)
    i.map = f.inv
    simple_graph_iso_new_inv(f.dst, f.src, f.inv, f.map, i)
    i.inv = f.map
}

/// The inverse graph isomorphism has the original inverse as forward map.
theorem simple_graph_iso_symm_map[V, W](f: SimpleGraphIso[V, W]) {
    simple_graph_iso_symm(f).map = f.inv
}

/// The inverse graph isomorphism has the original forward map as inverse.
theorem simple_graph_iso_symm_inv[V, W](f: SimpleGraphIso[V, W]) {
    simple_graph_iso_symm(f).inv = f.map
}

/// Construction of a graph homomorphism remembers the source graph.
theorem simple_graph_hom_new_src[V, W](
    g: SimpleGraph[V],
    h: SimpleGraph[W],
    f: V -> W,
    p: SimpleGraphHom[V, W]
) {
    SimpleGraphHom[V, W].new(g, h, f) = Option.some(p) implies p.src = g
}

/// Construction of a graph homomorphism remembers the target graph.
theorem simple_graph_hom_new_dst[V, W](
    g: SimpleGraph[V],
    h: SimpleGraph[W],
    f: V -> W,
    p: SimpleGraphHom[V, W]
) {
    SimpleGraphHom[V, W].new(g, h, f) = Option.some(p) implies p.dst = h
}

/// Construction of a graph homomorphism remembers the vertex map.
theorem simple_graph_hom_new_map[V, W](
    g: SimpleGraph[V],
    h: SimpleGraph[W],
    f: V -> W,
    p: SimpleGraphHom[V, W]
) {
    SimpleGraphHom[V, W].new(g, h, f) = Option.some(p) implies p.map = f
}

/// Existence of the restricted homomorphism from an induced source subgraph to its image graph.
theorem simple_graph_hom_induced_image_exists[V, W](f: SimpleGraphHom[V, W], s: Set[V]) {
    exists(h: SimpleGraphHom[V, W]) {
        h.src = induced_subgraph(f.src, s) and h.dst = simple_graph_hom_image(f, s) and h.map = f.map
    }
} by {
    simple_graph_hom_maps_induced_to_image(f, s)
    is_graph_hom(induced_subgraph(f.src, s), simple_graph_hom_image(f, s), f.map)
    let h: SimpleGraphHom[V, W] satisfy {
        SimpleGraphHom[V, W].new(induced_subgraph(f.src, s), simple_graph_hom_image(f, s), f.map) =
            Option.some(h)
    }
    simple_graph_hom_new_src(induced_subgraph(f.src, s), simple_graph_hom_image(f, s), f.map, h)
    h.src = induced_subgraph(f.src, s)
    simple_graph_hom_new_dst(induced_subgraph(f.src, s), simple_graph_hom_image(f, s), f.map, h)
    h.dst = simple_graph_hom_image(f, s)
    simple_graph_hom_new_map(induced_subgraph(f.src, s), simple_graph_hom_image(f, s), f.map, h)
    h.map = f.map
    h.src = induced_subgraph(f.src, s) and h.dst = simple_graph_hom_image(f, s) and h.map = f.map
}

/// Existence of the restricted homomorphism from a preimage graph to an induced target subgraph.
theorem simple_graph_hom_preimage_induced_exists[V, W](f: SimpleGraphHom[V, W], t: Set[W]) {
    exists(h: SimpleGraphHom[V, W]) {
        h.src = simple_graph_hom_preimage(f, t) and h.dst = induced_subgraph(f.dst, t) and h.map = f.map
    }
} by {
    simple_graph_hom_maps_preimage_to_induced(f, t)
    is_graph_hom(simple_graph_hom_preimage(f, t), induced_subgraph(f.dst, t), f.map)
    let h: SimpleGraphHom[V, W] satisfy {
        SimpleGraphHom[V, W].new(simple_graph_hom_preimage(f, t), induced_subgraph(f.dst, t), f.map) =
            Option.some(h)
    }
    simple_graph_hom_new_src(simple_graph_hom_preimage(f, t), induced_subgraph(f.dst, t), f.map, h)
    h.src = simple_graph_hom_preimage(f, t)
    simple_graph_hom_new_dst(simple_graph_hom_preimage(f, t), induced_subgraph(f.dst, t), f.map, h)
    h.dst = induced_subgraph(f.dst, t)
    simple_graph_hom_new_map(simple_graph_hom_preimage(f, t), induced_subgraph(f.dst, t), f.map, h)
    h.map = f.map
    h.src = simple_graph_hom_preimage(f, t) and h.dst = induced_subgraph(f.dst, t) and h.map = f.map
}

/// Construction of a graph embedding remembers the source graph.
theorem simple_graph_embedding_new_src[V, W](
    g: SimpleGraph[V],
    h: SimpleGraph[W],
    f: V -> W,
    p: SimpleGraphEmbedding[V, W]
) {
    SimpleGraphEmbedding[V, W].new(g, h, f) = Option.some(p) implies p.src = g
}

/// Construction of a graph embedding remembers the target graph.
theorem simple_graph_embedding_new_dst[V, W](
    g: SimpleGraph[V],
    h: SimpleGraph[W],
    f: V -> W,
    p: SimpleGraphEmbedding[V, W]
) {
    SimpleGraphEmbedding[V, W].new(g, h, f) = Option.some(p) implies p.dst = h
}

/// Construction of a graph embedding remembers the vertex map.
theorem simple_graph_embedding_new_map[V, W](
    g: SimpleGraph[V],
    h: SimpleGraph[W],
    f: V -> W,
    p: SimpleGraphEmbedding[V, W]
) {
    SimpleGraphEmbedding[V, W].new(g, h, f) = Option.some(p) implies p.map = f
}

/// Any vertex map is a graph homomorphism between empty graphs.
theorem empty_graph_any_fn_is_graph_hom[V, W](f: V -> W) {
    is_graph_hom(empty_graph[V], empty_graph[W], f)
} by {
    forall(x: V, y: V) {
        empty_graph_adj_false[V](x, y)
        not empty_graph[V].adj(x, y)
    }
}

/// Existence of the composite graph homomorphism when middle graphs match.
theorem simple_graph_hom_compose_exists[U, V, W](
    f: SimpleGraphHom[V, W],
    g: SimpleGraphHom[U, V]
) {
    f.src = g.dst implies exists(h: SimpleGraphHom[U, W]) {
        h.src = g.src and h.dst = f.dst and h.map = compose(f.map, g.map)
    }
} by {
    if f.src = g.dst {
        SimpleGraphHom.constraint(f.src, f.dst, f.map)
        is_graph_hom(f.src, f.dst, f.map)
        SimpleGraphHom.constraint(g.src, g.dst, g.map)
        is_graph_hom(g.src, g.dst, g.map)
        g.dst = f.src
        is_graph_hom(g.src, f.src, g.map)
        is_graph_hom_compose(g.src, f.src, f.dst, f.map, g.map)
        is_graph_hom(g.src, f.dst, compose(f.map, g.map))
        let h: SimpleGraphHom[U, W] satisfy {
            SimpleGraphHom[U, W].new(g.src, f.dst, compose(f.map, g.map)) = Option.some(h)
        }
        simple_graph_hom_new_src(g.src, f.dst, compose(f.map, g.map), h)
        simple_graph_hom_new_dst(g.src, f.dst, compose(f.map, g.map), h)
        simple_graph_hom_new_map(g.src, f.dst, compose(f.map, g.map), h)
        h.src = g.src and h.dst = f.dst and h.map = compose(f.map, g.map)
    }
}

/// First inverse for the composition of inverse vertex map pairs.
theorem inverse_vertex_maps_compose_left[U, V, W](
    f1: V -> W,
    e1: W -> V,
    f2: U -> V,
    e2: V -> U,
    x: U
) {
    (forall(a: V) { e1(f1(a)) = a }) and (forall(a: U) { e2(f2(a)) = a }) implies
        compose(e2, e1, compose(f1, f2, x)) = x
} by {
    if (forall(a: V) { e1(f1(a)) = a }) and (forall(a: U) { e2(f2(a)) = a }) {
        e1(f1(f2(x))) = f2(x)
        e2(f2(x)) = x
        compose(f1, f2, x) = f1(f2(x))
        compose(e2, e1, f1(f2(x))) = e2(e1(f1(f2(x))))
        compose(e2, e1, f1(f2(x))) = e2(f2(x))
        compose(e2, e1, f1(f2(x))) = x
    }
}

/// Second inverse for the composition of inverse vertex map pairs.
theorem inverse_vertex_maps_compose_right[U, V, W](
    f1: V -> W,
    e1: W -> V,
    f2: U -> V,
    e2: V -> U,
    y: W
) {
    (forall(a: V) { f2(e2(a)) = a }) and (forall(a: W) { f1(e1(a)) = a }) implies
        compose(f1, f2, compose(e2, e1, y)) = y
} by {
    if (forall(a: V) { f2(e2(a)) = a }) and (forall(a: W) { f1(e1(a)) = a }) {
        f1(e1(y)) = y
        f2(e2(e1(y))) = e1(y)
        compose(e2, e1, y) = e2(e1(y))
        compose(f1, f2, e2(e1(y))) = f1(f2(e2(e1(y))))
        compose(f1, f2, e2(e1(y))) = f1(e1(y))
        compose(f1, f2, e2(e1(y))) = y
    }
}

/// Composition of two inverse vertex map pairs is again an inverse vertex map pair.
theorem are_inverse_vertex_maps_compose[U, V, W](
    f1: V -> W,
    e1: W -> V,
    f2: U -> V,
    e2: V -> U
) {
    are_inverse_vertex_maps(f1, e1) and are_inverse_vertex_maps(f2, e2) implies
        are_inverse_vertex_maps(compose(f1, f2), compose(e2, e1))
} by {
    if are_inverse_vertex_maps(f1, e1) and are_inverse_vertex_maps(f2, e2) {
        are_inverse_vertex_maps(f1, e1) =
            ((forall(a: V) { e1(f1(a)) = a }) and (forall(b: W) { f1(e1(b)) = b }))
        are_inverse_vertex_maps(f2, e2) =
            ((forall(a: U) { e2(f2(a)) = a }) and (forall(b: V) { f2(e2(b)) = b }))
        (forall(a: V) { e1(f1(a)) = a })
        (forall(b: W) { f1(e1(b)) = b })
        (forall(a: U) { e2(f2(a)) = a })
        (forall(b: V) { f2(e2(b)) = b })
        (forall(a: V) { e1(f1(a)) = a }) and (forall(a: U) { e2(f2(a)) = a })
        (forall(a: V) { f2(e2(a)) = a }) and (forall(a: W) { f1(e1(a)) = a })
        let p: U -> W = compose(f1, f2)
        let q: W -> U = compose(e2, e1)
        forall(x: U) {
            inverse_vertex_maps_compose_left(f1, e1, f2, e2, x)
            compose(e2, e1, compose(f1, f2, x)) = x
            q(p(x)) = x
        }
        let h1: Bool = forall(x: U) { q(p(x)) = x }
        h1
        forall(y: W) {
            inverse_vertex_maps_compose_right(f1, e1, f2, e2, y)
            compose(f1, f2, compose(e2, e1, y)) = y
            p(q(y)) = y
        }
        let h2: Bool = forall(y: W) { p(q(y)) = y }
        h2
        h1 and h2
        are_inverse_vertex_maps(p, q)
        are_inverse_vertex_maps(compose(f1, f2), compose(e2, e1))
    }
}

/// Composition of two graph isomorphism pairs is a graph isomorphism pair.
theorem is_graph_iso_pair_compose[U, V, W](
    g: SimpleGraph[U],
    h: SimpleGraph[V],
    k: SimpleGraph[W],
    f1: V -> W,
    e1: W -> V,
    f2: U -> V,
    e2: V -> U
) {
    is_graph_iso_pair(h, k, f1, e1) and is_graph_iso_pair(g, h, f2, e2) implies
        is_graph_iso_pair(g, k, compose(f1, f2), compose(e2, e1))
} by {
    if is_graph_iso_pair(h, k, f1, e1) and is_graph_iso_pair(g, h, f2, e2) {
        is_graph_iso_pair(h, k, f1, e1) =
            (is_graph_hom(h, k, f1) and is_graph_hom(k, h, e1)
                and are_inverse_vertex_maps(f1, e1))
        is_graph_iso_pair(g, h, f2, e2) =
            (is_graph_hom(g, h, f2) and is_graph_hom(h, g, e2)
                and are_inverse_vertex_maps(f2, e2))
        is_graph_hom_compose(g, h, k, f1, f2)
        is_graph_hom(g, k, compose(f1, f2))
        is_graph_hom_compose(k, h, g, e2, e1)
        is_graph_hom(k, g, compose(e2, e1))
        are_inverse_vertex_maps_compose(f1, e1, f2, e2)
        are_inverse_vertex_maps(compose(f1, f2), compose(e2, e1))
        is_graph_iso_pair(g, k, compose(f1, f2), compose(e2, e1)) =
            (is_graph_hom(g, k, compose(f1, f2)) and is_graph_hom(k, g, compose(e2, e1))
                and are_inverse_vertex_maps(compose(f1, f2), compose(e2, e1)))
        is_graph_iso_pair(g, k, compose(f1, f2), compose(e2, e1))
    }
}

/// Existence of the composite graph isomorphism when middle graphs match.
theorem simple_graph_iso_compose_exists[U, V, W](
    f: SimpleGraphIso[V, W],
    g: SimpleGraphIso[U, V]
) {
    f.src = g.dst implies exists(h: SimpleGraphIso[U, W]) {
        h.src = g.src and h.dst = f.dst and h.map = compose(f.map, g.map) and h.inv = compose(g.inv, f.inv)
    }
} by {
    if f.src = g.dst {
        SimpleGraphIso.constraint(f.src, f.dst, f.map, f.inv)
        is_graph_iso_pair(f.src, f.dst, f.map, f.inv)
        SimpleGraphIso.constraint(g.src, g.dst, g.map, g.inv)
        is_graph_iso_pair(g.src, g.dst, g.map, g.inv)
        g.dst = f.src
        is_graph_iso_pair(g.src, f.src, g.map, g.inv)
        is_graph_iso_pair_compose(g.src, f.src, f.dst, f.map, f.inv, g.map, g.inv)
        is_graph_iso_pair(g.src, f.dst, compose(f.map, g.map), compose(g.inv, f.inv))
        let h: SimpleGraphIso[U, W] satisfy {
            SimpleGraphIso[U, W].new(g.src, f.dst, compose(f.map, g.map), compose(g.inv, f.inv))
                = Option.some(h)
        }
        simple_graph_iso_new_src(g.src, f.dst, compose(f.map, g.map), compose(g.inv, f.inv), h)
        simple_graph_iso_new_dst(g.src, f.dst, compose(f.map, g.map), compose(g.inv, f.inv), h)
        simple_graph_iso_new_map(g.src, f.dst, compose(f.map, g.map), compose(g.inv, f.inv), h)
        simple_graph_iso_new_inv(g.src, f.dst, compose(f.map, g.map), compose(g.inv, f.inv), h)
        h.src = g.src and h.dst = f.dst and h.map = compose(f.map, g.map) and h.inv = compose(g.inv, f.inv)
    }
}

/// Composition of two adjacency-reflecting vertex maps reflects adjacency.
theorem reflects_graph_adj_compose[U, V, W](
    g: SimpleGraph[U],
    h: SimpleGraph[V],
    k: SimpleGraph[W],
    f: V -> W,
    e: U -> V
) {
    reflects_graph_adj(h, k, f) and reflects_graph_adj(g, h, e) implies
        reflects_graph_adj(g, k, compose(f, e))
} by {
    if reflects_graph_adj(h, k, f) and reflects_graph_adj(g, h, e) {
        reflects_graph_adj(h, k, f) =
            forall(a: V, b: V) {
                k.adj(f(a), f(b)) implies h.adj(a, b)
            }
        reflects_graph_adj(g, h, e) =
            forall(a: U, b: U) {
                h.adj(e(a), e(b)) implies g.adj(a, b)
            }
        forall(x: U, y: U) {
            if k.adj(compose(f, e, x), compose(f, e, y)) {
                compose(f, e, x) = f(e(x))
                compose(f, e, y) = f(e(y))
                k.adj(f(e(x)), f(e(y)))
                reflects_graph_adj_apply(h, k, f, e(x), e(y))
                h.adj(e(x), e(y))
                reflects_graph_adj_apply(g, h, e, x, y)
                g.adj(x, y)
            }
        }
    }
}

/// Composition of two graph embeddings is a graph embedding.
theorem is_graph_embedding_compose[U, V, W](
    g: SimpleGraph[U],
    h: SimpleGraph[V],
    k: SimpleGraph[W],
    f: V -> W,
    e: U -> V
) {
    is_graph_embedding(h, k, f) and is_graph_embedding(g, h, e) implies
        is_graph_embedding(g, k, compose(f, e))
} by {
    if is_graph_embedding(h, k, f) and is_graph_embedding(g, h, e) {
        is_graph_embedding(h, k, f) =
            (is_graph_hom(h, k, f) and reflects_graph_adj(h, k, f) and is_injective_fn(f))
        is_graph_embedding(g, h, e) =
            (is_graph_hom(g, h, e) and reflects_graph_adj(g, h, e) and is_injective_fn(e))
        is_graph_hom(h, k, f)
        is_graph_hom(g, h, e)
        reflects_graph_adj(h, k, f)
        reflects_graph_adj(g, h, e)
        is_injective_fn(f)
        is_injective_fn(e)
        is_graph_hom_compose(g, h, k, f, e)
        is_graph_hom(g, k, compose(f, e))
        reflects_graph_adj_compose(g, h, k, f, e)
        reflects_graph_adj(g, k, compose(f, e))
        compose_injective_fn(f, e)
        is_injective_fn(compose(f, e))
        is_graph_embedding(g, k, compose(f, e)) =
            (is_graph_hom(g, k, compose(f, e)) and reflects_graph_adj(g, k, compose(f, e))
                and is_injective_fn(compose(f, e)))
        is_graph_embedding(g, k, compose(f, e))
    }
}

/// Existence of the composite graph embedding when middle graphs match.
theorem simple_graph_embedding_compose_exists[U, V, W](
    f: SimpleGraphEmbedding[V, W],
    g: SimpleGraphEmbedding[U, V]
) {
    f.src = g.dst implies exists(h: SimpleGraphEmbedding[U, W]) {
        h.src = g.src and h.dst = f.dst and h.map = compose(f.map, g.map)
    }
} by {
    if f.src = g.dst {
        SimpleGraphEmbedding.constraint(f.src, f.dst, f.map)
        is_graph_embedding(f.src, f.dst, f.map)
        SimpleGraphEmbedding.constraint(g.src, g.dst, g.map)
        is_graph_embedding(g.src, g.dst, g.map)
        g.dst = f.src
        is_graph_embedding(g.src, f.src, g.map)
        is_graph_embedding_compose(g.src, f.src, f.dst, f.map, g.map)
        is_graph_embedding(g.src, f.dst, compose(f.map, g.map))
        let h: SimpleGraphEmbedding[U, W] satisfy {
            SimpleGraphEmbedding[U, W].new(g.src, f.dst, compose(f.map, g.map)) = Option.some(h)
        }
        simple_graph_embedding_new_src(g.src, f.dst, compose(f.map, g.map), h)
        simple_graph_embedding_new_dst(g.src, f.dst, compose(f.map, g.map), h)
        simple_graph_embedding_new_map(g.src, f.dst, compose(f.map, g.map), h)
        h.src = g.src and h.dst = f.dst and h.map = compose(f.map, g.map)
    }
}

/// The image of a vertex set under the vertex map of a graph embedding.
define simple_graph_embedding_image_set[V, W](f: SimpleGraphEmbedding[V, W], s: Set[V]) -> Set[W] {
    set_image(s, f.map)
}

/// The preimage of a vertex set under the vertex map of a graph embedding.
define simple_graph_embedding_preimage_set[V, W](f: SimpleGraphEmbedding[V, W], t: Set[W]) -> Set[V] {
    set_preimage(f.map, t)
}

/// The image graph of an induced source subgraph under a graph embedding.
define simple_graph_embedding_image[V, W](f: SimpleGraphEmbedding[V, W], s: Set[V]) -> SimpleGraph[W] {
    induced_subgraph(f.dst, simple_graph_embedding_image_set(f, s))
}

/// The preimage graph of an induced target subgraph under a graph embedding.
define simple_graph_embedding_preimage[V, W](f: SimpleGraphEmbedding[V, W], t: Set[W]) -> SimpleGraph[V] {
    induced_subgraph(f.src, simple_graph_embedding_preimage_set(f, t))
}

/// The image set of a graph embedding is the direct image of the vertex set.
theorem simple_graph_embedding_image_set_eq[V, W](f: SimpleGraphEmbedding[V, W], s: Set[V]) {
    simple_graph_embedding_image_set(f, s) = set_image(s, f.map)
}

/// The preimage set of a graph embedding is the inverse image of the vertex set.
theorem simple_graph_embedding_preimage_set_eq[V, W](f: SimpleGraphEmbedding[V, W], t: Set[W]) {
    simple_graph_embedding_preimage_set(f, t) = set_preimage(f.map, t)
}

/// A vertex in the source set maps into the image set under a graph embedding.
theorem simple_graph_embedding_image_set_contains_map[V, W](
    f: SimpleGraphEmbedding[V, W], s: Set[V], x: V
) {
    s.contains(x) implies simple_graph_embedding_image_set(f, s).contains(f.map(x))
} by {
    if s.contains(x) {
        simple_graph_embedding_image_set_eq(f, s)
        maps_into_set_image(s, f.map, x)
        set_image(s, f.map).contains(f.map(x))
    }
}

/// A vertex lies in the embedding preimage set exactly when its image lies in the target set.
theorem simple_graph_embedding_preimage_set_contains_eq[V, W](
    f: SimpleGraphEmbedding[V, W], t: Set[W], x: V
) {
    simple_graph_embedding_preimage_set(f, t).contains(x) = t.contains(f.map(x))
} by {
    simple_graph_embedding_preimage_set_eq(f, t)
    set_preimage_contains_eq(f.map, t, x)
    simple_graph_embedding_preimage_set(f, t).contains(x) = set_preimage(f.map, t).contains(x)
    set_preimage(f.map, t).contains(x) = t.contains(f.map(x))
}

/// The image graph of an embedding is the induced subgraph on the image vertex set.
theorem simple_graph_embedding_image_eq[V, W](f: SimpleGraphEmbedding[V, W], s: Set[V]) {
    simple_graph_embedding_image(f, s) = induced_subgraph(f.dst, simple_graph_embedding_image_set(f, s))
}

/// The preimage graph of an embedding is the induced subgraph on the preimage vertex set.
theorem simple_graph_embedding_preimage_eq[V, W](f: SimpleGraphEmbedding[V, W], t: Set[W]) {
    simple_graph_embedding_preimage(f, t) = induced_subgraph(f.src, simple_graph_embedding_preimage_set(f, t))
}

/// Adjacency in the embedding image graph is adjacency with both endpoints in the image set.
theorem simple_graph_embedding_image_adj_eq[V, W](
    f: SimpleGraphEmbedding[V, W], s: Set[V], x: W, y: W
) {
    simple_graph_embedding_image(f, s).adj(x, y) =
        (f.dst.adj(x, y) and simple_graph_embedding_image_set(f, s).contains(x)
            and simple_graph_embedding_image_set(f, s).contains(y))
} by {
    simple_graph_embedding_image_eq(f, s)
    induced_subgraph_adj_eq(f.dst, simple_graph_embedding_image_set(f, s), x, y)
}

/// Adjacency in the embedding preimage graph is source adjacency with both images in the target set.
theorem simple_graph_embedding_preimage_adj_eq[V, W](
    f: SimpleGraphEmbedding[V, W], t: Set[W], x: V, y: V
) {
    simple_graph_embedding_preimage(f, t).adj(x, y) =
        (f.src.adj(x, y) and t.contains(f.map(x)) and t.contains(f.map(y)))
} by {
    simple_graph_embedding_preimage_eq(f, t)
    induced_subgraph_adj_eq(f.src, simple_graph_embedding_preimage_set(f, t), x, y)
    simple_graph_embedding_preimage_set_contains_eq(f, t, x)
    simple_graph_embedding_preimage_set_contains_eq(f, t, y)
}

/// The embedding maps the induced source subgraph into its image graph.
theorem simple_graph_embedding_maps_induced_to_image[V, W](
    f: SimpleGraphEmbedding[V, W], s: Set[V]
) {
    is_graph_hom(induced_subgraph(f.src, s), simple_graph_embedding_image(f, s), f.map)
} by {
    forall(x: V, y: V) {
        if induced_subgraph(f.src, s).adj(x, y) {
            induced_subgraph_adj_imp_base_adj(f.src, s, x, y)
            f.src.adj(x, y)
            simple_graph_embedding_maps_adj(f, x, y)
            f.dst.adj(f.map(x), f.map(y))
            induced_subgraph_adj_imp_left_mem(f.src, s, x, y)
            s.contains(x)
            induced_subgraph_adj_imp_right_mem(f.src, s, x, y)
            s.contains(y)
            simple_graph_embedding_image_set_contains_map(f, s, x)
            simple_graph_embedding_image_set(f, s).contains(f.map(x))
            simple_graph_embedding_image_set_contains_map(f, s, y)
            simple_graph_embedding_image_set(f, s).contains(f.map(y))
            simple_graph_embedding_image_adj_eq(f, s, f.map(x), f.map(y))
            simple_graph_embedding_image(f, s).adj(f.map(x), f.map(y))
        }
    }
}

/// The embedding maps the preimage graph into the induced target subgraph.
theorem simple_graph_embedding_maps_preimage_to_induced[V, W](
    f: SimpleGraphEmbedding[V, W], t: Set[W]
) {
    is_graph_hom(simple_graph_embedding_preimage(f, t), induced_subgraph(f.dst, t), f.map)
} by {
    forall(x: V, y: V) {
        if simple_graph_embedding_preimage(f, t).adj(x, y) {
            simple_graph_embedding_preimage_adj_eq(f, t, x, y)
            f.src.adj(x, y)
            t.contains(f.map(x))
            t.contains(f.map(y))
            simple_graph_embedding_maps_adj(f, x, y)
            f.dst.adj(f.map(x), f.map(y))
            induced_subgraph_adj_of_base_adj(f.dst, t, f.map(x), f.map(y))
            induced_subgraph(f.dst, t).adj(f.map(x), f.map(y))
        }
    }
}

/// The embedding map reflects adjacency from its image graph to the induced source subgraph.
theorem simple_graph_embedding_reflects_induced_to_image[V, W](
    f: SimpleGraphEmbedding[V, W], s: Set[V]
) {
    reflects_graph_adj(induced_subgraph(f.src, s), simple_graph_embedding_image(f, s), f.map)
} by {
    simple_graph_embedding_is_injective(f)
    is_injective_fn(f.map) = forall(a: V, b: V) {
        f.map(a) = f.map(b) implies a = b
    }
    forall(x: V, y: V) {
        if simple_graph_embedding_image(f, s).adj(f.map(x), f.map(y)) {
            simple_graph_embedding_image_adj_eq(f, s, f.map(x), f.map(y))
            f.dst.adj(f.map(x), f.map(y))
            simple_graph_embedding_image_set(f, s).contains(f.map(x))
            simple_graph_embedding_image_set(f, s).contains(f.map(y))
            simple_graph_embedding_image_set_eq(f, s)
            set_image(s, f.map).contains(f.map(x))
            set_image(s, f.map).contains(f.map(y))
            set_image_contains_eq(s, f.map, f.map(x))
            image_contains(f.map, s, f.map(x))
            image_contains(f.map, s, f.map(x)) = exists(a: V) {
                s.contains(a) and f.map(x) = f.map(a)
            }
            let a: V satisfy { s.contains(a) and f.map(x) = f.map(a) }
            f.map(a) = f.map(x)
            a = x
            s.contains(x)
            set_image_contains_eq(s, f.map, f.map(y))
            image_contains(f.map, s, f.map(y))
            image_contains(f.map, s, f.map(y)) = exists(b: V) {
                s.contains(b) and f.map(y) = f.map(b)
            }
            let b: V satisfy { s.contains(b) and f.map(y) = f.map(b) }
            f.map(b) = f.map(y)
            b = y
            s.contains(y)
            simple_graph_embedding_map_adj_iff(f, x, y)
            f.src.adj(x, y)
            induced_subgraph_adj_of_base_adj(f.src, s, x, y)
            induced_subgraph(f.src, s).adj(x, y)
        }
    }
}

/// The embedding map reflects adjacency from the induced target subgraph to the preimage graph.
theorem simple_graph_embedding_reflects_preimage_to_induced[V, W](
    f: SimpleGraphEmbedding[V, W], t: Set[W]
) {
    reflects_graph_adj(simple_graph_embedding_preimage(f, t), induced_subgraph(f.dst, t), f.map)
} by {
    forall(x: V, y: V) {
        if induced_subgraph(f.dst, t).adj(f.map(x), f.map(y)) {
            induced_subgraph_adj_imp_base_adj(f.dst, t, f.map(x), f.map(y))
            f.dst.adj(f.map(x), f.map(y))
            induced_subgraph_adj_imp_left_mem(f.dst, t, f.map(x), f.map(y))
            t.contains(f.map(x))
            induced_subgraph_adj_imp_right_mem(f.dst, t, f.map(x), f.map(y))
            t.contains(f.map(y))
            simple_graph_embedding_map_adj_iff(f, x, y)
            f.src.adj(x, y)
            simple_graph_embedding_preimage_adj_eq(f, t, x, y)
            simple_graph_embedding_preimage(f, t).adj(x, y)
        }
    }
}

/// The embedding restricts to a graph embedding from the induced source subgraph to its image graph.
theorem is_graph_embedding_induced_to_image[V, W](
    f: SimpleGraphEmbedding[V, W], s: Set[V]
) {
    is_graph_embedding(induced_subgraph(f.src, s), simple_graph_embedding_image(f, s), f.map)
} by {
    simple_graph_embedding_maps_induced_to_image(f, s)
    is_graph_hom(induced_subgraph(f.src, s), simple_graph_embedding_image(f, s), f.map)
    simple_graph_embedding_reflects_induced_to_image(f, s)
    reflects_graph_adj(induced_subgraph(f.src, s), simple_graph_embedding_image(f, s), f.map)
    simple_graph_embedding_is_injective(f)
    is_injective_fn(f.map)
    is_graph_embedding(induced_subgraph(f.src, s), simple_graph_embedding_image(f, s), f.map) =
        (is_graph_hom(induced_subgraph(f.src, s), simple_graph_embedding_image(f, s), f.map)
            and reflects_graph_adj(induced_subgraph(f.src, s), simple_graph_embedding_image(f, s), f.map)
            and is_injective_fn(f.map))
}

/// The embedding restricts to a graph embedding from the preimage graph to the induced target subgraph.
theorem is_graph_embedding_preimage_to_induced[V, W](
    f: SimpleGraphEmbedding[V, W], t: Set[W]
) {
    is_graph_embedding(simple_graph_embedding_preimage(f, t), induced_subgraph(f.dst, t), f.map)
} by {
    simple_graph_embedding_maps_preimage_to_induced(f, t)
    is_graph_hom(simple_graph_embedding_preimage(f, t), induced_subgraph(f.dst, t), f.map)
    simple_graph_embedding_reflects_preimage_to_induced(f, t)
    reflects_graph_adj(simple_graph_embedding_preimage(f, t), induced_subgraph(f.dst, t), f.map)
    simple_graph_embedding_is_injective(f)
    is_injective_fn(f.map)
    is_graph_embedding(simple_graph_embedding_preimage(f, t), induced_subgraph(f.dst, t), f.map) =
        (is_graph_hom(simple_graph_embedding_preimage(f, t), induced_subgraph(f.dst, t), f.map)
            and reflects_graph_adj(simple_graph_embedding_preimage(f, t), induced_subgraph(f.dst, t), f.map)
            and is_injective_fn(f.map))
}

/// Existence of the restricted graph embedding from an induced source subgraph to its image graph.
theorem simple_graph_embedding_induced_image_exists[V, W](
    f: SimpleGraphEmbedding[V, W], s: Set[V]
) {
    exists(h: SimpleGraphEmbedding[V, W]) {
        h.src = induced_subgraph(f.src, s) and h.dst = simple_graph_embedding_image(f, s)
            and h.map = f.map
    }
} by {
    is_graph_embedding_induced_to_image(f, s)
    is_graph_embedding(induced_subgraph(f.src, s), simple_graph_embedding_image(f, s), f.map)
    let h: SimpleGraphEmbedding[V, W] satisfy {
        SimpleGraphEmbedding[V, W].new(induced_subgraph(f.src, s), simple_graph_embedding_image(f, s), f.map)
            = Option.some(h)
    }
    simple_graph_embedding_new_src(induced_subgraph(f.src, s), simple_graph_embedding_image(f, s), f.map, h)
    simple_graph_embedding_new_dst(induced_subgraph(f.src, s), simple_graph_embedding_image(f, s), f.map, h)
    simple_graph_embedding_new_map(induced_subgraph(f.src, s), simple_graph_embedding_image(f, s), f.map, h)
}

/// Existence of the restricted graph embedding from a preimage graph to an induced target subgraph.
theorem simple_graph_embedding_preimage_induced_exists[V, W](
    f: SimpleGraphEmbedding[V, W], t: Set[W]
) {
    exists(h: SimpleGraphEmbedding[V, W]) {
        h.src = simple_graph_embedding_preimage(f, t) and h.dst = induced_subgraph(f.dst, t)
            and h.map = f.map
    }
} by {
    is_graph_embedding_preimage_to_induced(f, t)
    is_graph_embedding(simple_graph_embedding_preimage(f, t), induced_subgraph(f.dst, t), f.map)
    let h: SimpleGraphEmbedding[V, W] satisfy {
        SimpleGraphEmbedding[V, W].new(simple_graph_embedding_preimage(f, t), induced_subgraph(f.dst, t), f.map)
            = Option.some(h)
    }
    simple_graph_embedding_new_src(simple_graph_embedding_preimage(f, t), induced_subgraph(f.dst, t), f.map, h)
    simple_graph_embedding_new_dst(simple_graph_embedding_preimage(f, t), induced_subgraph(f.dst, t), f.map, h)
    simple_graph_embedding_new_map(simple_graph_embedding_preimage(f, t), induced_subgraph(f.dst, t), f.map, h)
}

/// The image of a vertex set under the forward map of a graph isomorphism.
define simple_graph_iso_image_set[V, W](f: SimpleGraphIso[V, W], s: Set[V]) -> Set[W] {
    set_image(s, f.map)
}

/// The image graph of an induced source subgraph under a graph isomorphism.
define simple_graph_iso_image[V, W](f: SimpleGraphIso[V, W], s: Set[V]) -> SimpleGraph[W] {
    induced_subgraph(f.dst, simple_graph_iso_image_set(f, s))
}

/// The iso image set is the direct image of the vertex set.
theorem simple_graph_iso_image_set_eq[V, W](f: SimpleGraphIso[V, W], s: Set[V]) {
    simple_graph_iso_image_set(f, s) = set_image(s, f.map)
}

/// The iso image graph is the induced subgraph on the image vertex set.
theorem simple_graph_iso_image_eq[V, W](f: SimpleGraphIso[V, W], s: Set[V]) {
    simple_graph_iso_image(f, s) = induced_subgraph(f.dst, simple_graph_iso_image_set(f, s))
}

/// A vertex in the source set maps into the iso image set.
theorem simple_graph_iso_image_set_contains_map[V, W](
    f: SimpleGraphIso[V, W], s: Set[V], x: V
) {
    s.contains(x) implies simple_graph_iso_image_set(f, s).contains(f.map(x))
} by {
    if s.contains(x) {
        simple_graph_iso_image_set_eq(f, s)
        maps_into_set_image(s, f.map, x)
        set_image(s, f.map).contains(f.map(x))
    }
}

/// Adjacency in the iso image graph is target adjacency with both endpoints in the image set.
theorem simple_graph_iso_image_adj_eq[V, W](
    f: SimpleGraphIso[V, W], s: Set[V], x: W, y: W
) {
    simple_graph_iso_image(f, s).adj(x, y) =
        (f.dst.adj(x, y) and simple_graph_iso_image_set(f, s).contains(x)
            and simple_graph_iso_image_set(f, s).contains(y))
} by {
    simple_graph_iso_image_eq(f, s)
    induced_subgraph_adj_eq(f.dst, simple_graph_iso_image_set(f, s), x, y)
}

/// The forward map of an iso restricts to a homomorphism from the induced source subgraph to its image graph.
theorem simple_graph_iso_maps_induced_to_image[V, W](
    f: SimpleGraphIso[V, W], s: Set[V]
) {
    is_graph_hom(induced_subgraph(f.src, s), simple_graph_iso_image(f, s), f.map)
} by {
    simple_graph_iso_map_is_hom(f)
    is_graph_hom(f.src, f.dst, f.map) = forall(a: V, b: V) {
        f.src.adj(a, b) implies f.dst.adj(f.map(a), f.map(b))
    }
    forall(x: V, y: V) {
        if induced_subgraph(f.src, s).adj(x, y) {
            induced_subgraph_adj_imp_base_adj(f.src, s, x, y)
            f.src.adj(x, y)
            f.dst.adj(f.map(x), f.map(y))
            induced_subgraph_adj_imp_left_mem(f.src, s, x, y)
            s.contains(x)
            induced_subgraph_adj_imp_right_mem(f.src, s, x, y)
            s.contains(y)
            simple_graph_iso_image_set_contains_map(f, s, x)
            simple_graph_iso_image_set(f, s).contains(f.map(x))
            simple_graph_iso_image_set_contains_map(f, s, y)
            simple_graph_iso_image_set(f, s).contains(f.map(y))
            simple_graph_iso_image_adj_eq(f, s, f.map(x), f.map(y))
            simple_graph_iso_image(f, s).adj(f.map(x), f.map(y))
        }
    }
}

/// The inverse map of an iso restricts to a homomorphism from the iso image graph back to the induced source subgraph.
theorem simple_graph_iso_inv_maps_image_to_induced[V, W](
    f: SimpleGraphIso[V, W], s: Set[V]
) {
    is_graph_hom(simple_graph_iso_image(f, s), induced_subgraph(f.src, s), f.inv)
} by {
    simple_graph_iso_inv_is_hom(f)
    is_graph_hom(f.dst, f.src, f.inv) = forall(a: W, b: W) {
        f.dst.adj(a, b) implies f.src.adj(f.inv(a), f.inv(b))
    }
    forall(x: W, y: W) {
        if simple_graph_iso_image(f, s).adj(x, y) {
            simple_graph_iso_image_adj_eq(f, s, x, y)
            f.dst.adj(x, y)
            simple_graph_iso_image_set(f, s).contains(x)
            simple_graph_iso_image_set(f, s).contains(y)
            simple_graph_iso_image_set_eq(f, s)
            set_image(s, f.map).contains(x)
            set_image(s, f.map).contains(y)
            set_image_contains_eq(s, f.map, x)
            image_contains(f.map, s, x)
            image_contains(f.map, s, x) = exists(a: V) {
                s.contains(a) and x = f.map(a)
            }
            let a: V satisfy { s.contains(a) and x = f.map(a) }
            simple_graph_iso_left_inv(f, a)
            f.inv(f.map(a)) = a
            f.inv(x) = a
            s.contains(f.inv(x))
            set_image_contains_eq(s, f.map, y)
            image_contains(f.map, s, y)
            image_contains(f.map, s, y) = exists(b: V) {
                s.contains(b) and y = f.map(b)
            }
            let b: V satisfy { s.contains(b) and y = f.map(b) }
            simple_graph_iso_left_inv(f, b)
            f.inv(f.map(b)) = b
            f.inv(y) = b
            s.contains(f.inv(y))
            f.src.adj(f.inv(x), f.inv(y))
            induced_subgraph_adj_of_base_adj(f.src, s, f.inv(x), f.inv(y))
            induced_subgraph(f.src, s).adj(f.inv(x), f.inv(y))
        }
    }
}

/// The forward and inverse maps of a graph isomorphism form an iso pair between the induced source subgraph and its image graph.
theorem is_graph_iso_pair_induced_to_image[V, W](
    f: SimpleGraphIso[V, W], s: Set[V]
) {
    is_graph_iso_pair(induced_subgraph(f.src, s), simple_graph_iso_image(f, s), f.map, f.inv)
} by {
    simple_graph_iso_maps_induced_to_image(f, s)
    is_graph_hom(induced_subgraph(f.src, s), simple_graph_iso_image(f, s), f.map)
    simple_graph_iso_inv_maps_image_to_induced(f, s)
    is_graph_hom(simple_graph_iso_image(f, s), induced_subgraph(f.src, s), f.inv)
    simple_graph_iso_inverse_maps(f)
    are_inverse_vertex_maps(f.map, f.inv)
    is_graph_iso_pair(induced_subgraph(f.src, s), simple_graph_iso_image(f, s), f.map, f.inv) =
        (is_graph_hom(induced_subgraph(f.src, s), simple_graph_iso_image(f, s), f.map)
            and is_graph_hom(simple_graph_iso_image(f, s), induced_subgraph(f.src, s), f.inv)
            and are_inverse_vertex_maps(f.map, f.inv))
}

/// Existence of the restricted graph isomorphism from an induced source subgraph to its image graph.
theorem simple_graph_iso_induced_image_exists[V, W](
    f: SimpleGraphIso[V, W], s: Set[V]
) {
    exists(h: SimpleGraphIso[V, W]) {
        h.src = induced_subgraph(f.src, s) and h.dst = simple_graph_iso_image(f, s)
            and h.map = f.map and h.inv = f.inv
    }
} by {
    is_graph_iso_pair_induced_to_image(f, s)
    let h: SimpleGraphIso[V, W] satisfy {
        SimpleGraphIso[V, W].new(induced_subgraph(f.src, s), simple_graph_iso_image(f, s), f.map, f.inv)
            = Option.some(h)
    }
    simple_graph_iso_new_src(induced_subgraph(f.src, s), simple_graph_iso_image(f, s), f.map, f.inv, h)
    simple_graph_iso_new_dst(induced_subgraph(f.src, s), simple_graph_iso_image(f, s), f.map, f.inv, h)
    simple_graph_iso_new_map(induced_subgraph(f.src, s), simple_graph_iso_image(f, s), f.map, f.inv, h)
    simple_graph_iso_new_inv(induced_subgraph(f.src, s), simple_graph_iso_image(f, s), f.map, f.inv, h)
}

/// The preimage of a vertex set under the forward map of a graph isomorphism.
define simple_graph_iso_preimage_set[V, W](f: SimpleGraphIso[V, W], t: Set[W]) -> Set[V] {
    set_preimage(f.map, t)
}

/// The preimage graph of an induced target subgraph under a graph isomorphism.
define simple_graph_iso_preimage[V, W](f: SimpleGraphIso[V, W], t: Set[W]) -> SimpleGraph[V] {
    induced_subgraph(f.src, simple_graph_iso_preimage_set(f, t))
}

/// The iso preimage set is the inverse image of the vertex set.
theorem simple_graph_iso_preimage_set_eq[V, W](f: SimpleGraphIso[V, W], t: Set[W]) {
    simple_graph_iso_preimage_set(f, t) = set_preimage(f.map, t)
}

/// The iso preimage graph is the induced subgraph on the preimage vertex set.
theorem simple_graph_iso_preimage_eq[V, W](f: SimpleGraphIso[V, W], t: Set[W]) {
    simple_graph_iso_preimage(f, t) = induced_subgraph(f.src, simple_graph_iso_preimage_set(f, t))
}

/// A vertex lies in the iso preimage set exactly when its image lies in the target set.
theorem simple_graph_iso_preimage_set_contains_eq[V, W](
    f: SimpleGraphIso[V, W], t: Set[W], x: V
) {
    simple_graph_iso_preimage_set(f, t).contains(x) = t.contains(f.map(x))
} by {
    simple_graph_iso_preimage_set_eq(f, t)
    set_preimage_contains_eq(f.map, t, x)
    simple_graph_iso_preimage_set(f, t).contains(x) = set_preimage(f.map, t).contains(x)
    set_preimage(f.map, t).contains(x) = t.contains(f.map(x))
}

/// Adjacency in the iso preimage graph is source adjacency with both images in the target set.
theorem simple_graph_iso_preimage_adj_eq[V, W](
    f: SimpleGraphIso[V, W], t: Set[W], x: V, y: V
) {
    simple_graph_iso_preimage(f, t).adj(x, y) =
        (f.src.adj(x, y) and t.contains(f.map(x)) and t.contains(f.map(y)))
} by {
    simple_graph_iso_preimage_eq(f, t)
    induced_subgraph_adj_eq(f.src, simple_graph_iso_preimage_set(f, t), x, y)
    simple_graph_iso_preimage_set_contains_eq(f, t, x)
    simple_graph_iso_preimage_set_contains_eq(f, t, y)
}

/// The forward map of an iso restricts to a homomorphism from the preimage graph to the induced target subgraph.
theorem simple_graph_iso_maps_preimage_to_induced[V, W](
    f: SimpleGraphIso[V, W], t: Set[W]
) {
    is_graph_hom(simple_graph_iso_preimage(f, t), induced_subgraph(f.dst, t), f.map)
} by {
    simple_graph_iso_map_is_hom(f)
    is_graph_hom(f.src, f.dst, f.map) = forall(a: V, b: V) {
        f.src.adj(a, b) implies f.dst.adj(f.map(a), f.map(b))
    }
    forall(x: V, y: V) {
        if simple_graph_iso_preimage(f, t).adj(x, y) {
            simple_graph_iso_preimage_adj_eq(f, t, x, y)
            f.src.adj(x, y)
            t.contains(f.map(x))
            t.contains(f.map(y))
            f.dst.adj(f.map(x), f.map(y))
            induced_subgraph_adj_of_base_adj(f.dst, t, f.map(x), f.map(y))
            induced_subgraph(f.dst, t).adj(f.map(x), f.map(y))
        }
    }
}

/// The inverse map of an iso restricts to a homomorphism from the induced target subgraph back to the preimage graph.
theorem simple_graph_iso_inv_maps_induced_to_preimage[V, W](
    f: SimpleGraphIso[V, W], t: Set[W]
) {
    is_graph_hom(induced_subgraph(f.dst, t), simple_graph_iso_preimage(f, t), f.inv)
} by {
    simple_graph_iso_inv_is_hom(f)
    is_graph_hom(f.dst, f.src, f.inv) = forall(a: W, b: W) {
        f.dst.adj(a, b) implies f.src.adj(f.inv(a), f.inv(b))
    }
    forall(x: W, y: W) {
        if induced_subgraph(f.dst, t).adj(x, y) {
            induced_subgraph_adj_imp_base_adj(f.dst, t, x, y)
            f.dst.adj(x, y)
            induced_subgraph_adj_imp_left_mem(f.dst, t, x, y)
            t.contains(x)
            induced_subgraph_adj_imp_right_mem(f.dst, t, x, y)
            t.contains(y)
            f.src.adj(f.inv(x), f.inv(y))
            simple_graph_iso_right_inv(f, x)
            f.map(f.inv(x)) = x
            t.contains(f.map(f.inv(x)))
            simple_graph_iso_preimage_set_contains_eq(f, t, f.inv(x))
            simple_graph_iso_preimage_set(f, t).contains(f.inv(x))
            simple_graph_iso_right_inv(f, y)
            f.map(f.inv(y)) = y
            t.contains(f.map(f.inv(y)))
            simple_graph_iso_preimage_set_contains_eq(f, t, f.inv(y))
            simple_graph_iso_preimage_set(f, t).contains(f.inv(y))
            simple_graph_iso_preimage_adj_eq(f, t, f.inv(x), f.inv(y))
            simple_graph_iso_preimage(f, t).adj(f.inv(x), f.inv(y))
        }
    }
}

/// The forward and inverse maps of a graph isomorphism form an iso pair between the preimage graph and the induced target subgraph.
theorem is_graph_iso_pair_preimage_to_induced[V, W](
    f: SimpleGraphIso[V, W], t: Set[W]
) {
    is_graph_iso_pair(simple_graph_iso_preimage(f, t), induced_subgraph(f.dst, t), f.map, f.inv)
} by {
    simple_graph_iso_maps_preimage_to_induced(f, t)
    is_graph_hom(simple_graph_iso_preimage(f, t), induced_subgraph(f.dst, t), f.map)
    simple_graph_iso_inv_maps_induced_to_preimage(f, t)
    is_graph_hom(induced_subgraph(f.dst, t), simple_graph_iso_preimage(f, t), f.inv)
    simple_graph_iso_inverse_maps(f)
    are_inverse_vertex_maps(f.map, f.inv)
    is_graph_iso_pair(simple_graph_iso_preimage(f, t), induced_subgraph(f.dst, t), f.map, f.inv) =
        (is_graph_hom(simple_graph_iso_preimage(f, t), induced_subgraph(f.dst, t), f.map)
            and is_graph_hom(induced_subgraph(f.dst, t), simple_graph_iso_preimage(f, t), f.inv)
            and are_inverse_vertex_maps(f.map, f.inv))
}

/// Existence of the restricted graph isomorphism from a preimage graph to an induced target subgraph.
theorem simple_graph_iso_preimage_induced_exists[V, W](
    f: SimpleGraphIso[V, W], t: Set[W]
) {
    exists(h: SimpleGraphIso[V, W]) {
        h.src = simple_graph_iso_preimage(f, t) and h.dst = induced_subgraph(f.dst, t)
            and h.map = f.map and h.inv = f.inv
    }
} by {
    is_graph_iso_pair_preimage_to_induced(f, t)
    let h: SimpleGraphIso[V, W] satisfy {
        SimpleGraphIso[V, W].new(simple_graph_iso_preimage(f, t), induced_subgraph(f.dst, t), f.map, f.inv)
            = Option.some(h)
    }
    simple_graph_iso_new_src(simple_graph_iso_preimage(f, t), induced_subgraph(f.dst, t), f.map, f.inv, h)
    simple_graph_iso_new_dst(simple_graph_iso_preimage(f, t), induced_subgraph(f.dst, t), f.map, f.inv, h)
    simple_graph_iso_new_map(simple_graph_iso_preimage(f, t), induced_subgraph(f.dst, t), f.map, f.inv, h)
    simple_graph_iso_new_inv(simple_graph_iso_preimage(f, t), induced_subgraph(f.dst, t), f.map, f.inv, h)
}

/// The identity map reflects adjacency of any graph.
theorem identity_fn_reflects_graph_adj[V](g: SimpleGraph[V]) {
    reflects_graph_adj(g, g, identity_fn[V])
} by {
    forall(x: V, y: V) {
        if g.adj(identity_fn[V](x), identity_fn[V](y)) {
            identity_fn[V](x) = x
            identity_fn[V](y) = y
            g.adj(x, y)
        }
    }
}

/// The identity map is injective.
theorem identity_fn_is_injective[V](g: SimpleGraph[V]) {
    is_injective_fn(identity_fn[V])
} by {
    forall(x: V, y: V) {
        if identity_fn[V](x) = identity_fn[V](y) {
            identity_fn[V](x) = x
            identity_fn[V](y) = y
            x = y
        }
    }
}

/// The identity map is a graph embedding.
theorem identity_fn_is_graph_embedding[V](g: SimpleGraph[V]) {
    is_graph_embedding(g, g, identity_fn[V])
} by {
    identity_fn_is_graph_hom(g)
    identity_fn_reflects_graph_adj(g)
    identity_fn_is_injective(g)
    is_graph_embedding(g, g, identity_fn[V]) =
        (is_graph_hom(g, g, identity_fn[V]) and reflects_graph_adj(g, g, identity_fn[V])
            and is_injective_fn(identity_fn[V]))
}

/// The identity embedding of a simple graph.
let simple_graph_embedding_refl[V](g: SimpleGraph[V]) -> result: SimpleGraphEmbedding[V, V] satisfy {
    result.src = g and result.dst = g and result.map = identity_fn[V]
} by {
    identity_fn_is_graph_embedding(g)
    let e: SimpleGraphEmbedding[V, V] satisfy {
        SimpleGraphEmbedding[V, V].new(g, g, identity_fn[V]) = Option.some(e)
    }
    simple_graph_embedding_new_src(g, g, identity_fn[V], e)
    simple_graph_embedding_new_dst(g, g, identity_fn[V], e)
    simple_graph_embedding_new_map(g, g, identity_fn[V], e)
}

/// The identity embedding has the identity vertex map.
theorem simple_graph_embedding_refl_map[V](g: SimpleGraph[V]) {
    simple_graph_embedding_refl(g).map = identity_fn[V]
}

/// The identity embedding has the original graph as its source.
theorem simple_graph_embedding_refl_src[V](g: SimpleGraph[V]) {
    simple_graph_embedding_refl(g).src = g
}

/// The identity embedding has the original graph as its target.
theorem simple_graph_embedding_refl_dst[V](g: SimpleGraph[V]) {
    simple_graph_embedding_refl(g).dst = g
}

/// The forward map of a graph isomorphism pair reflects adjacency.
theorem is_graph_iso_pair_reflects_adj[V, W](
    g: SimpleGraph[V], h: SimpleGraph[W], f: V -> W, e: W -> V
) {
    is_graph_iso_pair(g, h, f, e) implies reflects_graph_adj(g, h, f)
} by {
    if is_graph_iso_pair(g, h, f, e) {
        is_graph_iso_pair(g, h, f, e) =
            (is_graph_hom(g, h, f) and is_graph_hom(h, g, e)
                and are_inverse_vertex_maps(f, e))
        is_graph_hom(h, g, e)
        is_graph_hom(h, g, e) =
            forall(a: W, b: W) {
                h.adj(a, b) implies g.adj(e(a), e(b))
            }
        are_inverse_vertex_maps(f, e) =
            ((forall(x: V) { e(f(x)) = x }) and (forall(y: W) { f(e(y)) = y }))
        forall(x: V, y: V) {
            if h.adj(f(x), f(y)) {
                g.adj(e(f(x)), e(f(y)))
                e(f(x)) = x
                e(f(y)) = y
                g.adj(x, y)
            }
        }
        reflects_graph_adj(g, h, f) =
            forall(x: V, y: V) {
                h.adj(f(x), f(y)) implies g.adj(x, y)
            }
        forall(x: V, y: V) {
            h.adj(f(x), f(y)) implies g.adj(x, y)
        }
        (forall(x: V, y: V) { h.adj(f(x), f(y)) implies g.adj(x, y) }) = true
        reflects_graph_adj(g, h, f) = true
        reflects_graph_adj(g, h, f)
    }
}

/// The forward map of a graph isomorphism pair is injective.
theorem is_graph_iso_pair_map_is_injective[V, W](
    g: SimpleGraph[V], h: SimpleGraph[W], f: V -> W, e: W -> V
) {
    is_graph_iso_pair(g, h, f, e) implies is_injective_fn(f)
} by {
    if is_graph_iso_pair(g, h, f, e) {
        is_graph_iso_pair(g, h, f, e) =
            (is_graph_hom(g, h, f) and is_graph_hom(h, g, e)
                and are_inverse_vertex_maps(f, e))
        are_inverse_vertex_maps(f, e) =
            ((forall(x: V) { e(f(x)) = x }) and (forall(y: W) { f(e(y)) = y }))
        forall(x: V) {
            e(f(x)) = x
        }
        is_left_inverse_fn(f, e)
        left_inverse_fn_imp_injective_fn(f, e)
    }
}

/// The forward map of a graph isomorphism is a graph embedding.
theorem is_graph_iso_pair_is_graph_embedding[V, W](
    g: SimpleGraph[V], h: SimpleGraph[W], f: V -> W, e: W -> V
) {
    is_graph_iso_pair(g, h, f, e) implies is_graph_embedding(g, h, f)
} by {
    if is_graph_iso_pair(g, h, f, e) {
        is_graph_iso_pair(g, h, f, e) =
            (is_graph_hom(g, h, f) and is_graph_hom(h, g, e)
                and are_inverse_vertex_maps(f, e))
        is_graph_hom(g, h, f) and is_graph_hom(h, g, e) and are_inverse_vertex_maps(f, e)
        is_graph_hom(g, h, f)
        is_graph_iso_pair_reflects_adj(g, h, f, e)
        is_graph_iso_pair_map_is_injective(g, h, f, e)
        is_graph_hom(g, h, f) and reflects_graph_adj(g, h, f) and is_injective_fn(f)
        is_graph_embedding(g, h, f) =
            (is_graph_hom(g, h, f) and reflects_graph_adj(g, h, f) and is_injective_fn(f))
        is_graph_embedding(g, h, f)
    }
}

/// A graph isomorphism gives rise to a graph embedding with the same source, target, and forward map.
let simple_graph_embedding_of_iso[V, W](f: SimpleGraphIso[V, W]) -> result: SimpleGraphEmbedding[V, W] satisfy {
    result.src = f.src and result.dst = f.dst and result.map = f.map
} by {
    SimpleGraphIso.constraint(f.src, f.dst, f.map, f.inv)
    is_graph_iso_pair(f.src, f.dst, f.map, f.inv)
    is_graph_iso_pair_is_graph_embedding(f.src, f.dst, f.map, f.inv)
    is_graph_embedding(f.src, f.dst, f.map)
    let e: SimpleGraphEmbedding[V, W] satisfy {
        SimpleGraphEmbedding[V, W].new(f.src, f.dst, f.map) = Option.some(e)
    }
    simple_graph_embedding_new_src(f.src, f.dst, f.map, e)
    simple_graph_embedding_new_dst(f.src, f.dst, f.map, e)
    simple_graph_embedding_new_map(f.src, f.dst, f.map, e)
}

/// The embedding induced by an isomorphism has the same forward map.
theorem simple_graph_embedding_of_iso_map[V, W](f: SimpleGraphIso[V, W]) {
    simple_graph_embedding_of_iso(f).map = f.map
}

/// The embedding induced by an isomorphism has the same source.
theorem simple_graph_embedding_of_iso_src[V, W](f: SimpleGraphIso[V, W]) {
    simple_graph_embedding_of_iso(f).src = f.src
}

/// The embedding induced by an isomorphism has the same target.
theorem simple_graph_embedding_of_iso_dst[V, W](f: SimpleGraphIso[V, W]) {
    simple_graph_embedding_of_iso(f).dst = f.dst
}

/// An injective vertex map is a graph homomorphism from any graph into the complete graph.
theorem injective_fn_is_graph_hom_to_complete_graph[V, W](
    g: SimpleGraph[V],
    f: V -> W
) {
    is_injective_fn(f) implies is_graph_hom(g, complete_graph[W], f)
} by {
    if is_injective_fn(f) {
        is_injective_fn(f) =
            forall(a: V, b: V) {
                f(a) = f(b) implies a = b
            }
        forall(x: V, y: V) {
            if g.adj(x, y) {
                simple_graph_adj_ne(g, x, y)
                x != y
                if f(x) = f(y) {
                    x = y
                    false
                }
                f(x) != f(y)
                complete_graph_adj_iff_ne[W](f(x), f(y))
                complete_graph[W].adj(f(x), f(y))
            }
        }
    }
}

/// Existence of a graph homomorphism from any graph into the complete graph via an injective vertex map.
theorem simple_graph_hom_to_complete_graph_exists[V, W](
    g: SimpleGraph[V],
    f: V -> W
) {
    is_injective_fn(f) implies exists(p: SimpleGraphHom[V, W]) {
        p.src = g and p.dst = complete_graph[W] and p.map = f
    }
} by {
    if is_injective_fn(f) {
        injective_fn_is_graph_hom_to_complete_graph(g, f)
        is_graph_hom(g, complete_graph[W], f)
        let p: SimpleGraphHom[V, W] satisfy {
            SimpleGraphHom[V, W].new(g, complete_graph[W], f) = Option.some(p)
        }
        simple_graph_hom_new_src(g, complete_graph[W], f, p)
        simple_graph_hom_new_dst(g, complete_graph[W], f, p)
        simple_graph_hom_new_map(g, complete_graph[W], f, p)
        p.src = g and p.dst = complete_graph[W] and p.map = f
    }
}

/// An injective vertex map between complete graphs reflects adjacency.
theorem injective_fn_reflects_complete_graph_adj[V, W](f: V -> W) {
    is_injective_fn(f) implies reflects_graph_adj(complete_graph[V], complete_graph[W], f)
} by {
    if is_injective_fn(f) {
        is_injective_fn(f) =
            forall(a: V, b: V) {
                f(a) = f(b) implies a = b
            }
        forall(x: V, y: V) {
            if complete_graph[W].adj(f(x), f(y)) {
                complete_graph_adj_iff_ne[W](f(x), f(y))
                f(x) != f(y)
                if x = y {
                    f(x) = f(y)
                    false
                }
                x != y
                complete_graph_adj_iff_ne[V](x, y)
                complete_graph[V].adj(x, y)
            }
        }
    }
}

/// An injective vertex map is a graph embedding between complete graphs.
theorem injective_fn_is_graph_embedding_between_complete_graphs[V, W](f: V -> W) {
    is_injective_fn(f) implies is_graph_embedding(complete_graph[V], complete_graph[W], f)
} by {
    if is_injective_fn(f) {
        injective_fn_is_graph_hom_to_complete_graph(complete_graph[V], f)
        is_graph_hom(complete_graph[V], complete_graph[W], f)
        injective_fn_reflects_complete_graph_adj(f)
        reflects_graph_adj(complete_graph[V], complete_graph[W], f)
        (is_graph_hom(complete_graph[V], complete_graph[W], f)
            and reflects_graph_adj(complete_graph[V], complete_graph[W], f)
            and is_injective_fn(f))
        is_graph_embedding(complete_graph[V], complete_graph[W], f) =
            (is_graph_hom(complete_graph[V], complete_graph[W], f)
                and reflects_graph_adj(complete_graph[V], complete_graph[W], f)
                and is_injective_fn(f))
        is_graph_embedding(complete_graph[V], complete_graph[W], f)
    }
}

/// Existence of a graph embedding between complete graphs via an injective vertex map.
theorem simple_graph_embedding_between_complete_graphs_exists[V, W](f: V -> W) {
    is_injective_fn(f) implies exists(p: SimpleGraphEmbedding[V, W]) {
        p.src = complete_graph[V] and p.dst = complete_graph[W] and p.map = f
    }
} by {
    if is_injective_fn(f) {
        injective_fn_is_graph_embedding_between_complete_graphs(f)
        is_graph_embedding(complete_graph[V], complete_graph[W], f)
        let p: SimpleGraphEmbedding[V, W] satisfy {
            SimpleGraphEmbedding[V, W].new(complete_graph[V], complete_graph[W], f) = Option.some(p)
        }
        simple_graph_embedding_new_src(complete_graph[V], complete_graph[W], f, p)
        simple_graph_embedding_new_dst(complete_graph[V], complete_graph[W], f, p)
        simple_graph_embedding_new_map(complete_graph[V], complete_graph[W], f, p)
        p.src = complete_graph[V] and p.dst = complete_graph[W] and p.map = f
    }
}

/// The packaging map of a graph isomorphism remembers the source graph.
theorem simple_graph_iso_new_src_dst_map_inv_remembered[V, W](
    g: SimpleGraph[V],
    h: SimpleGraph[W],
    fwd: V -> W,
    rev: W -> V,
    i: SimpleGraphIso[V, W]
) {
    SimpleGraphIso[V, W].new(g, h, fwd, rev) = Option.some(i) implies
        (i.src = g and i.dst = h and i.map = fwd and i.inv = rev)
} by {
    if SimpleGraphIso[V, W].new(g, h, fwd, rev) = Option.some(i) {
        simple_graph_iso_new_src(g, h, fwd, rev, i)
        i.src = g
        simple_graph_iso_new_dst(g, h, fwd, rev, i)
        i.dst = h
        simple_graph_iso_new_map(g, h, fwd, rev, i)
        i.map = fwd
        simple_graph_iso_new_inv(g, h, fwd, rev, i)
        i.inv = rev
        i.src = g and i.dst = h and i.map = fwd and i.inv = rev
    }
}

/// The packaging map of a graph homomorphism remembers source, target, and map together.
theorem simple_graph_hom_new_src_dst_map_remembered[V, W](
    g: SimpleGraph[V],
    h: SimpleGraph[W],
    f: V -> W,
    p: SimpleGraphHom[V, W]
) {
    SimpleGraphHom[V, W].new(g, h, f) = Option.some(p) implies
        (p.src = g and p.dst = h and p.map = f)
} by {
    if SimpleGraphHom[V, W].new(g, h, f) = Option.some(p) {
        simple_graph_hom_new_src(g, h, f, p)
        simple_graph_hom_new_dst(g, h, f, p)
        simple_graph_hom_new_map(g, h, f, p)
    }
}

/// The packaging map of a graph embedding remembers source, target, and map together.
theorem simple_graph_embedding_new_src_dst_map_remembered[V, W](
    g: SimpleGraph[V],
    h: SimpleGraph[W],
    f: V -> W,
    p: SimpleGraphEmbedding[V, W]
) {
    SimpleGraphEmbedding[V, W].new(g, h, f) = Option.some(p) implies
        (p.src = g and p.dst = h and p.map = f)
} by {
    if SimpleGraphEmbedding[V, W].new(g, h, f) = Option.some(p) {
        simple_graph_embedding_new_src(g, h, f, p)
        simple_graph_embedding_new_dst(g, h, f, p)
        simple_graph_embedding_new_map(g, h, f, p)
    }
}

/// The complement of a union is the intersection of the complements.
theorem graph_complement_union[V](g: SimpleGraph[V], h: SimpleGraph[V]) {
    graph_complement(graph_union(g, h)) =
        graph_intersection(graph_complement(g), graph_complement(h))
} by {
    forall(x: V, y: V) {
        graph_complement_adj_eq(graph_union(g, h), x, y)
        graph_union_adj_eq(g, h, x, y)
        graph_intersection_adj_eq(graph_complement(g), graph_complement(h), x, y)
        graph_complement_adj_eq(g, x, y)
        graph_complement_adj_eq(h, x, y)
        if graph_complement(graph_union(g, h)).adj(x, y) {
            x != y
            not graph_union(g, h).adj(x, y)
            if g.adj(x, y) {
                graph_union(g, h).adj(x, y)
                false
            }
            if h.adj(x, y) {
                graph_union(g, h).adj(x, y)
                false
            }
            not g.adj(x, y)
            not h.adj(x, y)
            graph_complement(g).adj(x, y)
            graph_complement(h).adj(x, y)
            graph_intersection(graph_complement(g), graph_complement(h)).adj(x, y)
        }
        if graph_intersection(graph_complement(g), graph_complement(h)).adj(x, y) {
            graph_complement(g).adj(x, y)
            graph_complement(h).adj(x, y)
            x != y
            not g.adj(x, y)
            not h.adj(x, y)
            if graph_union(g, h).adj(x, y) {
                if g.adj(x, y) {
                    false
                } else {
                    h.adj(x, y)
                    false
                }
            }
            not graph_union(g, h).adj(x, y)
            graph_complement(graph_union(g, h)).adj(x, y)
        }
        graph_complement(graph_union(g, h)).adj(x, y) =
            graph_intersection(graph_complement(g), graph_complement(h)).adj(x, y)
    }
    simple_graph_ext(
        graph_complement(graph_union(g, h)),
        graph_intersection(graph_complement(g), graph_complement(h))
    )
}

/// The complement of an intersection is the union of the complements.
theorem graph_complement_intersection[V](g: SimpleGraph[V], h: SimpleGraph[V]) {
    graph_complement(graph_intersection(g, h)) =
        graph_union(graph_complement(g), graph_complement(h))
} by {
    forall(x: V, y: V) {
        graph_complement_adj_eq(graph_intersection(g, h), x, y)
        graph_intersection_adj_eq(g, h, x, y)
        graph_union_adj_eq(graph_complement(g), graph_complement(h), x, y)
        graph_complement_adj_eq(g, x, y)
        graph_complement_adj_eq(h, x, y)
        if graph_complement(graph_intersection(g, h)).adj(x, y) {
            x != y
            not graph_intersection(g, h).adj(x, y)
            if g.adj(x, y) {
                if h.adj(x, y) {
                    graph_intersection(g, h).adj(x, y)
                    false
                }
                not h.adj(x, y)
                graph_complement(h).adj(x, y)
                graph_union(graph_complement(g), graph_complement(h)).adj(x, y)
            } else {
                not g.adj(x, y)
                graph_complement(g).adj(x, y)
                graph_union(graph_complement(g), graph_complement(h)).adj(x, y)
            }
        }
        if graph_union(graph_complement(g), graph_complement(h)).adj(x, y) {
            if graph_complement(g).adj(x, y) {
                x != y
                not g.adj(x, y)
                if graph_intersection(g, h).adj(x, y) {
                    g.adj(x, y)
                    false
                }
                not graph_intersection(g, h).adj(x, y)
                graph_complement(graph_intersection(g, h)).adj(x, y)
            } else {
                graph_complement(h).adj(x, y)
                x != y
                not h.adj(x, y)
                if graph_intersection(g, h).adj(x, y) {
                    h.adj(x, y)
                    false
                }
                not graph_intersection(g, h).adj(x, y)
                graph_complement(graph_intersection(g, h)).adj(x, y)
            }
        }
        graph_complement(graph_intersection(g, h)).adj(x, y) =
            graph_union(graph_complement(g), graph_complement(h)).adj(x, y)
    }
    simple_graph_ext(
        graph_complement(graph_intersection(g, h)),
        graph_union(graph_complement(g), graph_complement(h))
    )
}
