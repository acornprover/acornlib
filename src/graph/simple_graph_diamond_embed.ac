from nat import Nat
from data.fin.fin import Fin
from data.basic.functions import is_injective_fn, injective_fn_eq
from finite_set import FiniteSet
from graph.simple_graph import SimpleGraph, simple_graph_adj_ne, simple_graph_adj_symmetric,
    is_graph_embedding, graph_embedding_is_injective
from graph.simple_graph_vertex_sets import common_neighborhood, common_neighborhood_contains_eq
from graph.simple_graph_diamond import has_nonadjacent_common_neighbors,
    has_nonadjacent_common_neighbors_intro, has_nonadjacent_common_neighbors_witness,
    is_diamond_free, is_diamond_free_intro, is_diamond_free_apply
from graph.simple_graph_diamond_graph import diamond4_graph, diamond4_graph_adj_iff
from graph.simple_graph_claw_embed import claw_map, claw_map_cases, claw_map_injective,
    four_distinct, claw4_value_cases
from graph.simple_graph_claw_converse import claw_witness_set, claw_witness_set_contains,
    claw4_witness, claw4_labels
from graph.simple_graph_forbidden import contains_induced, contains_induced_intro,
    contains_induced_witness, is_free_of, is_free_of_intro, is_graph_embedding_of_adj_eq,
    graph_embedding_adj_forward, graph_embedding_adj_backward

numerals Nat

/// The five edges and one non-edge of a diamond configuration.
///
/// Named so that the embedding statement is not ten conjuncts deep. The pair `a`, `b` is
/// adjacent, both are adjacent to each of `c` and `d`, and `c` and `d` are distinct and not
/// adjacent: exactly the complete graph on four vertices with the edge `c`-`d` removed.
define is_diamond_at[V](g: SimpleGraph[V], a: V, b: V, c: V, d: V) -> Bool {
    g.adj(a, b) and g.adj(a, c) and g.adj(a, d) and g.adj(b, c) and g.adj(b, d)
        and c != d and not g.adj(c, d)
}

/// The six conditions give a diamond configuration.
theorem is_diamond_at_intro[V](g: SimpleGraph[V], a: V, b: V, c: V, d: V) {
    g.adj(a, b) and g.adj(a, c) and g.adj(a, d) and g.adj(b, c) and g.adj(b, d)
        and c != d and not g.adj(c, d) implies is_diamond_at(g, a, b, c, d)
} by {
    if g.adj(a, b) and g.adj(a, c) and g.adj(a, d) and g.adj(b, c) and g.adj(b, d)
        and c != d and not g.adj(c, d) {
        (is_diamond_at(g, a, b, c, d)
            = (g.adj(a, b) and g.adj(a, c) and g.adj(a, d) and g.adj(b, c) and g.adj(b, d)
                and c != d and not g.adj(c, d)))
        is_diamond_at(g, a, b, c, d)
    }
}

/// The conditions can be read back off a diamond configuration.
theorem is_diamond_at_apply[V](g: SimpleGraph[V], a: V, b: V, c: V, d: V) {
    is_diamond_at(g, a, b, c, d) implies
        g.adj(a, b) and g.adj(a, c) and g.adj(a, d) and g.adj(b, c) and g.adj(b, d)
            and c != d and not g.adj(c, d)
} by {
    if is_diamond_at(g, a, b, c, d) {
        (is_diamond_at(g, a, b, c, d)
            = (g.adj(a, b) and g.adj(a, c) and g.adj(a, d) and g.adj(b, c) and g.adj(b, d)
                and c != d and not g.adj(c, d)))
        (g.adj(a, b) and g.adj(a, c) and g.adj(a, d) and g.adj(b, c) and g.adj(b, d)
            and c != d and not g.adj(c, d))
    }
}

/// The four vertices of a diamond configuration are pairwise distinct.
theorem diamond_four_distinct[V](g: SimpleGraph[V], a: V, b: V, c: V, d: V) {
    is_diamond_at(g, a, b, c, d) implies four_distinct(a, b, c, d)
} by {
    if is_diamond_at(g, a, b, c, d) {
        is_diamond_at_apply(g, a, b, c, d)
        (g.adj(a, b) and g.adj(a, c) and g.adj(a, d) and g.adj(b, c) and g.adj(b, d)
            and c != d and not g.adj(c, d))
        simple_graph_adj_ne(g, a, b)
        a != b
        simple_graph_adj_ne(g, a, c)
        a != c
        simple_graph_adj_ne(g, a, d)
        a != d
        simple_graph_adj_ne(g, b, c)
        b != c
        simple_graph_adj_ne(g, b, d)
        b != d
        (four_distinct(a, b, c, d)
            = (a != b and a != c and a != d and b != c and b != d and c != d))
        four_distinct(a, b, c, d)
    }
}

/// The adjacencies of a diamond configuration, in both orders and at repeats.
///
/// Sixteen facts, since the case analysis below meets every ordered pair of labels. Symmetry
/// supplies the reversed edges and irreflexivity the repeats, and proof search assembles
/// neither from the configuration on its own.
theorem diamond_pairs[V](g: SimpleGraph[V], a: V, b: V, c: V, d: V) {
    is_diamond_at(g, a, b, c, d) implies
        not g.adj(a, a)
            and g.adj(a, b)
            and g.adj(a, c)
            and g.adj(a, d)
            and g.adj(b, a)
            and not g.adj(b, b)
            and g.adj(b, c)
            and g.adj(b, d)
            and g.adj(c, a)
            and g.adj(c, b)
            and not g.adj(c, c)
            and not g.adj(c, d)
            and g.adj(d, a)
            and g.adj(d, b)
            and not g.adj(d, c)
            and not g.adj(d, d)
} by {
    if is_diamond_at(g, a, b, c, d) {
        is_diamond_at_apply(g, a, b, c, d)
        (g.adj(a, b) and g.adj(a, c) and g.adj(a, d) and g.adj(b, c) and g.adj(b, d)
            and c != d and not g.adj(c, d))
        simple_graph_adj_symmetric(g, a, b)
        (g.adj(a, b) = g.adj(b, a))
        simple_graph_adj_symmetric(g, a, c)
        (g.adj(a, c) = g.adj(c, a))
        simple_graph_adj_symmetric(g, a, d)
        (g.adj(a, d) = g.adj(d, a))
        simple_graph_adj_symmetric(g, b, c)
        (g.adj(b, c) = g.adj(c, b))
        simple_graph_adj_symmetric(g, b, d)
        (g.adj(b, d) = g.adj(d, b))
        simple_graph_adj_symmetric(g, c, d)
        (g.adj(c, d) = g.adj(d, c))
        if g.adj(a, a) {
            simple_graph_adj_ne(g, a, a)
            a != a
            false
        }
        not g.adj(a, a)
        if g.adj(b, b) {
            simple_graph_adj_ne(g, b, b)
            b != b
            false
        }
        not g.adj(b, b)
        if g.adj(c, c) {
            simple_graph_adj_ne(g, c, c)
            c != c
            false
        }
        not g.adj(c, c)
        if g.adj(d, d) {
            simple_graph_adj_ne(g, d, d)
            d != d
            false
        }
        not g.adj(d, d)
        not g.adj(a, a)
        g.adj(a, b)
        g.adj(a, c)
        g.adj(a, d)
        g.adj(b, a)
        not g.adj(b, b)
        g.adj(b, c)
        g.adj(b, d)
        g.adj(c, a)
        g.adj(c, b)
        not g.adj(c, c)
        not g.adj(c, d)
        g.adj(d, a)
        g.adj(d, b)
        not g.adj(d, c)
        not g.adj(d, d)
        (not g.adj(a, a)
            and g.adj(a, b)
            and g.adj(a, c)
            and g.adj(a, d)
            and g.adj(b, a)
            and not g.adj(b, b)
            and g.adj(b, c)
            and g.adj(b, d)
            and g.adj(c, a)
            and g.adj(c, b)
            and not g.adj(c, c)
            and not g.adj(c, d)
            and g.adj(d, a)
            and g.adj(d, b)
            and not g.adj(d, c)
            and not g.adj(d, d))
    }
}

/// A diamond configuration carries an induced diamond.
///
/// The map is the same four-point map the claw uses: label `i` goes to the `i`-th vertex of the
/// configuration. Only the adjacency it has to match differs.
theorem diamond_map_is_embedding[V](g: SimpleGraph[V], a: V, b: V, c: V, d: V) {
    is_diamond_at(g, a, b, c, d)
        implies is_graph_embedding(diamond4_graph, g, claw_map(a, b, c, d))
} by {
    if is_diamond_at(g, a, b, c, d) {
        diamond_four_distinct(g, a, b, c, d)
        four_distinct(a, b, c, d)
        claw_map_injective(a, b, c, d)
        is_injective_fn(claw_map(a, b, c, d))
        diamond_pairs(g, a, b, c, d)
        (not g.adj(a, a)
            and g.adj(a, b)
            and g.adj(a, c)
            and g.adj(a, d)
            and g.adj(b, a)
            and not g.adj(b, b)
            and g.adj(b, c)
            and g.adj(b, d)
            and g.adj(c, a)
            and g.adj(c, b)
            and not g.adj(c, c)
            and not g.adj(c, d)
            and g.adj(d, a)
            and g.adj(d, b)
            and not g.adj(d, c)
            and not g.adj(d, d))
        forall(x: Fin[Nat.4], y: Fin[Nat.4]) {
            diamond4_graph_adj_iff(x, y)
            (diamond4_graph.adj(x, y) = (x.value != y.value
                and (x.value = Nat.0 or x.value = Nat.1
                    or y.value = Nat.0 or y.value = Nat.1)))
            claw_map_cases(a, b, c, d, x)
            ((x.value = Nat.0 and claw_map(a, b, c, d)(x) = a)
                or (x.value = Nat.1 and claw_map(a, b, c, d)(x) = b)
                or (x.value = Nat.2 and claw_map(a, b, c, d)(x) = c)
                or (x.value = Nat.3 and claw_map(a, b, c, d)(x) = d))
            claw_map_cases(a, b, c, d, y)
            ((y.value = Nat.0 and claw_map(a, b, c, d)(y) = a)
                or (y.value = Nat.1 and claw_map(a, b, c, d)(y) = b)
                or (y.value = Nat.2 and claw_map(a, b, c, d)(y) = c)
                or (y.value = Nat.3 and claw_map(a, b, c, d)(y) = d))
            claw4_value_cases(x)
            (x.value = Nat.0 or x.value = Nat.1 or x.value = Nat.2 or x.value = Nat.3)
            claw4_value_cases(y)
            (y.value = Nat.0 or y.value = Nat.1 or y.value = Nat.2 or y.value = Nat.3)
            if x.value = Nat.0 {
                claw_map(a, b, c, d)(x) = a
                if y.value = Nat.0 {
                    claw_map(a, b, c, d)(y) = a
                    not g.adj(a, a)
                    not g.adj(claw_map(a, b, c, d)(x), claw_map(a, b, c, d)(y))
                    not diamond4_graph.adj(x, y)
                    (g.adj(claw_map(a, b, c, d)(x), claw_map(a, b, c, d)(y))
                        = diamond4_graph.adj(x, y))
                }
                if y.value = Nat.1 {
                    claw_map(a, b, c, d)(y) = b
                    g.adj(a, b)
                    g.adj(claw_map(a, b, c, d)(x), claw_map(a, b, c, d)(y))
                    diamond4_graph.adj(x, y)
                    (g.adj(claw_map(a, b, c, d)(x), claw_map(a, b, c, d)(y))
                        = diamond4_graph.adj(x, y))
                }
                if y.value = Nat.2 {
                    claw_map(a, b, c, d)(y) = c
                    g.adj(a, c)
                    g.adj(claw_map(a, b, c, d)(x), claw_map(a, b, c, d)(y))
                    diamond4_graph.adj(x, y)
                    (g.adj(claw_map(a, b, c, d)(x), claw_map(a, b, c, d)(y))
                        = diamond4_graph.adj(x, y))
                }
                if y.value = Nat.3 {
                    claw_map(a, b, c, d)(y) = d
                    g.adj(a, d)
                    g.adj(claw_map(a, b, c, d)(x), claw_map(a, b, c, d)(y))
                    diamond4_graph.adj(x, y)
                    (g.adj(claw_map(a, b, c, d)(x), claw_map(a, b, c, d)(y))
                        = diamond4_graph.adj(x, y))
                }
                (g.adj(claw_map(a, b, c, d)(x), claw_map(a, b, c, d)(y))
                    = diamond4_graph.adj(x, y))
            }
            if x.value = Nat.1 {
                claw_map(a, b, c, d)(x) = b
                if y.value = Nat.0 {
                    claw_map(a, b, c, d)(y) = a
                    g.adj(b, a)
                    g.adj(claw_map(a, b, c, d)(x), claw_map(a, b, c, d)(y))
                    diamond4_graph.adj(x, y)
                    (g.adj(claw_map(a, b, c, d)(x), claw_map(a, b, c, d)(y))
                        = diamond4_graph.adj(x, y))
                }
                if y.value = Nat.1 {
                    claw_map(a, b, c, d)(y) = b
                    not g.adj(b, b)
                    not g.adj(claw_map(a, b, c, d)(x), claw_map(a, b, c, d)(y))
                    not diamond4_graph.adj(x, y)
                    (g.adj(claw_map(a, b, c, d)(x), claw_map(a, b, c, d)(y))
                        = diamond4_graph.adj(x, y))
                }
                if y.value = Nat.2 {
                    claw_map(a, b, c, d)(y) = c
                    g.adj(b, c)
                    g.adj(claw_map(a, b, c, d)(x), claw_map(a, b, c, d)(y))
                    diamond4_graph.adj(x, y)
                    (g.adj(claw_map(a, b, c, d)(x), claw_map(a, b, c, d)(y))
                        = diamond4_graph.adj(x, y))
                }
                if y.value = Nat.3 {
                    claw_map(a, b, c, d)(y) = d
                    g.adj(b, d)
                    g.adj(claw_map(a, b, c, d)(x), claw_map(a, b, c, d)(y))
                    diamond4_graph.adj(x, y)
                    (g.adj(claw_map(a, b, c, d)(x), claw_map(a, b, c, d)(y))
                        = diamond4_graph.adj(x, y))
                }
                (g.adj(claw_map(a, b, c, d)(x), claw_map(a, b, c, d)(y))
                    = diamond4_graph.adj(x, y))
            }
            if x.value = Nat.2 {
                claw_map(a, b, c, d)(x) = c
                if y.value = Nat.0 {
                    claw_map(a, b, c, d)(y) = a
                    g.adj(c, a)
                    g.adj(claw_map(a, b, c, d)(x), claw_map(a, b, c, d)(y))
                    diamond4_graph.adj(x, y)
                    (g.adj(claw_map(a, b, c, d)(x), claw_map(a, b, c, d)(y))
                        = diamond4_graph.adj(x, y))
                }
                if y.value = Nat.1 {
                    claw_map(a, b, c, d)(y) = b
                    g.adj(c, b)
                    g.adj(claw_map(a, b, c, d)(x), claw_map(a, b, c, d)(y))
                    diamond4_graph.adj(x, y)
                    (g.adj(claw_map(a, b, c, d)(x), claw_map(a, b, c, d)(y))
                        = diamond4_graph.adj(x, y))
                }
                if y.value = Nat.2 {
                    claw_map(a, b, c, d)(y) = c
                    not g.adj(c, c)
                    not g.adj(claw_map(a, b, c, d)(x), claw_map(a, b, c, d)(y))
                    not diamond4_graph.adj(x, y)
                    (g.adj(claw_map(a, b, c, d)(x), claw_map(a, b, c, d)(y))
                        = diamond4_graph.adj(x, y))
                }
                if y.value = Nat.3 {
                    claw_map(a, b, c, d)(y) = d
                    not g.adj(c, d)
                    not g.adj(claw_map(a, b, c, d)(x), claw_map(a, b, c, d)(y))
                    not diamond4_graph.adj(x, y)
                    (g.adj(claw_map(a, b, c, d)(x), claw_map(a, b, c, d)(y))
                        = diamond4_graph.adj(x, y))
                }
                (g.adj(claw_map(a, b, c, d)(x), claw_map(a, b, c, d)(y))
                    = diamond4_graph.adj(x, y))
            }
            if x.value = Nat.3 {
                claw_map(a, b, c, d)(x) = d
                if y.value = Nat.0 {
                    claw_map(a, b, c, d)(y) = a
                    g.adj(d, a)
                    g.adj(claw_map(a, b, c, d)(x), claw_map(a, b, c, d)(y))
                    diamond4_graph.adj(x, y)
                    (g.adj(claw_map(a, b, c, d)(x), claw_map(a, b, c, d)(y))
                        = diamond4_graph.adj(x, y))
                }
                if y.value = Nat.1 {
                    claw_map(a, b, c, d)(y) = b
                    g.adj(d, b)
                    g.adj(claw_map(a, b, c, d)(x), claw_map(a, b, c, d)(y))
                    diamond4_graph.adj(x, y)
                    (g.adj(claw_map(a, b, c, d)(x), claw_map(a, b, c, d)(y))
                        = diamond4_graph.adj(x, y))
                }
                if y.value = Nat.2 {
                    claw_map(a, b, c, d)(y) = c
                    not g.adj(d, c)
                    not g.adj(claw_map(a, b, c, d)(x), claw_map(a, b, c, d)(y))
                    not diamond4_graph.adj(x, y)
                    (g.adj(claw_map(a, b, c, d)(x), claw_map(a, b, c, d)(y))
                        = diamond4_graph.adj(x, y))
                }
                if y.value = Nat.3 {
                    claw_map(a, b, c, d)(y) = d
                    not g.adj(d, d)
                    not g.adj(claw_map(a, b, c, d)(x), claw_map(a, b, c, d)(y))
                    not diamond4_graph.adj(x, y)
                    (g.adj(claw_map(a, b, c, d)(x), claw_map(a, b, c, d)(y))
                        = diamond4_graph.adj(x, y))
                }
                (g.adj(claw_map(a, b, c, d)(x), claw_map(a, b, c, d)(y))
                    = diamond4_graph.adj(x, y))
            }
            (g.adj(claw_map(a, b, c, d)(x), claw_map(a, b, c, d)(y))
                = diamond4_graph.adj(x, y))
        }
        is_graph_embedding_of_adj_eq(diamond4_graph, g, claw_map(a, b, c, d))
        is_graph_embedding(diamond4_graph, g, claw_map(a, b, c, d))
    }
}

/// A graph with a diamond configuration contains an induced diamond.
theorem contains_induced_diamond_of_config[V](g: SimpleGraph[V], a: V, b: V, c: V, d: V) {
    is_diamond_at(g, a, b, c, d) implies contains_induced(g, diamond4_graph)
} by {
    if is_diamond_at(g, a, b, c, d) {
        diamond_map_is_embedding(g, a, b, c, d)
        is_graph_embedding(diamond4_graph, g, claw_map(a, b, c, d))
        contains_induced_intro(g, diamond4_graph, claw_map(a, b, c, d))
        contains_induced(g, diamond4_graph)
    }
}

/// An adjacent pair with two non-adjacent common neighbors is a diamond configuration.
theorem is_diamond_at_of_common_neighbors[V](
    g: SimpleGraph[V], s: FiniteSet[V], x: V, y: V, z: V, w: V
) {
    g.adj(x, y) and common_neighborhood(g, s, x, y).contains(z)
        and common_neighborhood(g, s, x, y).contains(w)
        and z != w and not g.adj(z, w) implies is_diamond_at(g, x, y, z, w)
} by {
    if g.adj(x, y) and common_neighborhood(g, s, x, y).contains(z)
        and common_neighborhood(g, s, x, y).contains(w)
        and z != w and not g.adj(z, w) {
        common_neighborhood_contains_eq(g, s, x, y, z)
        (common_neighborhood(g, s, x, y).contains(z)
            = (s.contains(z) and g.adj(x, z) and g.adj(y, z)))
        g.adj(x, z)
        g.adj(y, z)
        common_neighborhood_contains_eq(g, s, x, y, w)
        (common_neighborhood(g, s, x, y).contains(w)
            = (s.contains(w) and g.adj(x, w) and g.adj(y, w)))
        g.adj(x, w)
        g.adj(y, w)
        is_diamond_at_intro(g, x, y, z, w)
        is_diamond_at(g, x, y, z, w)
    }
}

/// A graph free of the diamond is diamond-free on every vertex set.
theorem is_diamond_free_of_free_of_diamond[V](g: SimpleGraph[V], s: FiniteSet[V]) {
    is_free_of(g, diamond4_graph) implies is_diamond_free(g, s)
} by {
    if is_free_of(g, diamond4_graph) {
        (is_free_of(g, diamond4_graph) = not contains_induced(g, diamond4_graph))
        not contains_induced(g, diamond4_graph)
        forall(x: V, y: V) {
            if s.contains(x) and s.contains(y) and g.adj(x, y) {
                if has_nonadjacent_common_neighbors(g, s, x, y) {
                    has_nonadjacent_common_neighbors_witness(g, s, x, y)
                    exists(z: V, w: V) {
                        common_neighborhood(g, s, x, y).contains(z)
                            and common_neighborhood(g, s, x, y).contains(w)
                            and z != w and not g.adj(z, w)
                    }
                    let (p: V, q: V) satisfy {
                        common_neighborhood(g, s, x, y).contains(p)
                            and common_neighborhood(g, s, x, y).contains(q)
                            and p != q and not g.adj(p, q)
                    }
                    is_diamond_at_of_common_neighbors(g, s, x, y, p, q)
                    is_diamond_at(g, x, y, p, q)
                    contains_induced_diamond_of_config(g, x, y, p, q)
                    contains_induced(g, diamond4_graph)
                    false
                }
                not has_nonadjacent_common_neighbors(g, s, x, y)
            }
            (s.contains(x) and s.contains(y) and g.adj(x, y)
                implies not has_nonadjacent_common_neighbors(g, s, x, y))
        }
        is_diamond_free_intro(g, s)
        is_diamond_free(g, s)
    }
}

/// An induced diamond gives an adjacent pair with two non-adjacent common neighbors.
///
/// The four images of the embedding are the configuration, read off the same way as for the
/// claw: adjacent where the diamond is adjacent, non-adjacent where it is not, and distinct
/// because the embedding is injective.
theorem config_of_contains_induced_diamond[V](g: SimpleGraph[V]) {
    contains_induced(g, diamond4_graph) implies exists(s: FiniteSet[V], x: V, y: V) {
        s.contains(x) and s.contains(y) and g.adj(x, y)
            and has_nonadjacent_common_neighbors(g, s, x, y)
    }
} by {
    if contains_induced(g, diamond4_graph) {
        contains_induced_witness(g, diamond4_graph)
        exists(f: Fin[Nat.4] -> V) { is_graph_embedding(diamond4_graph, g, f) }
        let (m: Fin[Nat.4] -> V) satisfy {
            is_graph_embedding(diamond4_graph, g, m)
        }
        claw4_labels
        (Nat.0 < Nat.4 and Nat.1 < Nat.4 and Nat.2 < Nat.4 and Nat.3 < Nat.4
            and Nat.0 != Nat.1 and Nat.0 != Nat.2 and Nat.0 != Nat.3
            and Nat.1 != Nat.2 and Nat.1 != Nat.3 and Nat.2 != Nat.3)
        claw4_witness(Nat.0)
        exists(i: Fin[Nat.4]) { i.value = Nat.0 }
        let (v0: Fin[Nat.4]) satisfy {
            v0.value = Nat.0
        }
        claw4_witness(Nat.1)
        exists(i: Fin[Nat.4]) { i.value = Nat.1 }
        let (v1: Fin[Nat.4]) satisfy {
            v1.value = Nat.1
        }
        claw4_witness(Nat.2)
        exists(i: Fin[Nat.4]) { i.value = Nat.2 }
        let (v2: Fin[Nat.4]) satisfy {
            v2.value = Nat.2
        }
        claw4_witness(Nat.3)
        exists(i: Fin[Nat.4]) { i.value = Nat.3 }
        let (v3: Fin[Nat.4]) satisfy {
            v3.value = Nat.3
        }
        diamond4_graph_adj_iff(v0, v1)
        (diamond4_graph.adj(v0, v1) = (v0.value != v1.value
            and (v0.value = Nat.0 or v0.value = Nat.1
                or v1.value = Nat.0 or v1.value = Nat.1)))
        diamond4_graph.adj(v0, v1)
        graph_embedding_adj_forward(diamond4_graph, g, m, v0, v1)
        g.adj(m(v0), m(v1))
        diamond4_graph_adj_iff(v0, v2)
        (diamond4_graph.adj(v0, v2) = (v0.value != v2.value
            and (v0.value = Nat.0 or v0.value = Nat.1
                or v2.value = Nat.0 or v2.value = Nat.1)))
        diamond4_graph.adj(v0, v2)
        graph_embedding_adj_forward(diamond4_graph, g, m, v0, v2)
        g.adj(m(v0), m(v2))
        diamond4_graph_adj_iff(v0, v3)
        (diamond4_graph.adj(v0, v3) = (v0.value != v3.value
            and (v0.value = Nat.0 or v0.value = Nat.1
                or v3.value = Nat.0 or v3.value = Nat.1)))
        diamond4_graph.adj(v0, v3)
        graph_embedding_adj_forward(diamond4_graph, g, m, v0, v3)
        g.adj(m(v0), m(v3))
        diamond4_graph_adj_iff(v1, v2)
        (diamond4_graph.adj(v1, v2) = (v1.value != v2.value
            and (v1.value = Nat.0 or v1.value = Nat.1
                or v2.value = Nat.0 or v2.value = Nat.1)))
        diamond4_graph.adj(v1, v2)
        graph_embedding_adj_forward(diamond4_graph, g, m, v1, v2)
        g.adj(m(v1), m(v2))
        diamond4_graph_adj_iff(v1, v3)
        (diamond4_graph.adj(v1, v3) = (v1.value != v3.value
            and (v1.value = Nat.0 or v1.value = Nat.1
                or v3.value = Nat.0 or v3.value = Nat.1)))
        diamond4_graph.adj(v1, v3)
        graph_embedding_adj_forward(diamond4_graph, g, m, v1, v3)
        g.adj(m(v1), m(v3))
        diamond4_graph_adj_iff(v2, v3)
        (diamond4_graph.adj(v2, v3) = (v2.value != v3.value
            and (v2.value = Nat.0 or v2.value = Nat.1
                or v3.value = Nat.0 or v3.value = Nat.1)))
        Nat.2 != Nat.0
        Nat.2 != Nat.1
        Nat.3 != Nat.0
        Nat.3 != Nat.1
        v2.value != Nat.0
        v2.value != Nat.1
        v3.value != Nat.0
        v3.value != Nat.1
        not (v2.value = Nat.0 or v2.value = Nat.1
            or v3.value = Nat.0 or v3.value = Nat.1)
        not diamond4_graph.adj(v2, v3)
        if g.adj(m(v2), m(v3)) {
            graph_embedding_adj_backward(diamond4_graph, g, m, v2, v3)
            diamond4_graph.adj(v2, v3)
            false
        }
        not g.adj(m(v2), m(v3))
        graph_embedding_is_injective(diamond4_graph, g, m)
        is_injective_fn(m)
        if m(v2) = m(v3) {
            injective_fn_eq(m, v2, v3)
            v2 = v3
            (v2.value = v3.value)
            false
        }
        m(v2) != m(v3)
        claw_witness_set_contains(m(v0), m(v1), m(v2), m(v3))
        (claw_witness_set(m(v0), m(v1), m(v2), m(v3)).contains(m(v0))
            and claw_witness_set(m(v0), m(v1), m(v2), m(v3)).contains(m(v1))
            and claw_witness_set(m(v0), m(v1), m(v2), m(v3)).contains(m(v2))
            and claw_witness_set(m(v0), m(v1), m(v2), m(v3)).contains(m(v3)))
        common_neighborhood_contains_eq(g, claw_witness_set(m(v0), m(v1), m(v2), m(v3)),
            m(v0), m(v1), m(v2))
        (common_neighborhood(g, claw_witness_set(m(v0), m(v1), m(v2), m(v3)),
            m(v0), m(v1)).contains(m(v2))
            = (claw_witness_set(m(v0), m(v1), m(v2), m(v3)).contains(m(v2))
                and g.adj(m(v0), m(v2)) and g.adj(m(v1), m(v2))))
        common_neighborhood(g, claw_witness_set(m(v0), m(v1), m(v2), m(v3)),
            m(v0), m(v1)).contains(m(v2))
        common_neighborhood_contains_eq(g, claw_witness_set(m(v0), m(v1), m(v2), m(v3)),
            m(v0), m(v1), m(v3))
        (common_neighborhood(g, claw_witness_set(m(v0), m(v1), m(v2), m(v3)),
            m(v0), m(v1)).contains(m(v3))
            = (claw_witness_set(m(v0), m(v1), m(v2), m(v3)).contains(m(v3))
                and g.adj(m(v0), m(v3)) and g.adj(m(v1), m(v3))))
        common_neighborhood(g, claw_witness_set(m(v0), m(v1), m(v2), m(v3)),
            m(v0), m(v1)).contains(m(v3))
        has_nonadjacent_common_neighbors_intro(g,
            claw_witness_set(m(v0), m(v1), m(v2), m(v3)), m(v0), m(v1), m(v2), m(v3))
        has_nonadjacent_common_neighbors(g,
            claw_witness_set(m(v0), m(v1), m(v2), m(v3)), m(v0), m(v1))
        exists(s: FiniteSet[V], x: V, y: V) {
            s.contains(x) and s.contains(y) and g.adj(x, y)
                and has_nonadjacent_common_neighbors(g, s, x, y)
        }
    }
}

/// A graph diamond-free on every vertex set is free of the diamond.
theorem free_of_diamond_of_is_diamond_free[V](g: SimpleGraph[V]) {
    (forall(s: FiniteSet[V]) { is_diamond_free(g, s) })
        implies is_free_of(g, diamond4_graph)
} by {
    if forall(s: FiniteSet[V]) { is_diamond_free(g, s) } {
        if contains_induced(g, diamond4_graph) {
            config_of_contains_induced_diamond(g)
            exists(s: FiniteSet[V], x: V, y: V) {
                s.contains(x) and s.contains(y) and g.adj(x, y)
                    and has_nonadjacent_common_neighbors(g, s, x, y)
            }
            let (t: FiniteSet[V], p: V, q: V) satisfy {
                t.contains(p) and t.contains(q) and g.adj(p, q)
                    and has_nonadjacent_common_neighbors(g, t, p, q)
            }
            is_diamond_free(g, t)
            is_diamond_free_apply(g, t, p, q)
            (is_diamond_free(g, t) and t.contains(p) and t.contains(q) and g.adj(p, q)
                implies not has_nonadjacent_common_neighbors(g, t, p, q))
            not has_nonadjacent_common_neighbors(g, t, p, q)
            false
        }
        not contains_induced(g, diamond4_graph)
        is_free_of_intro(g, diamond4_graph)
        is_free_of(g, diamond4_graph)
    }
}

/// Freedom from the four-vertex diamond and diamond-freeness on every vertex set agree.
theorem free_of_diamond_iff_diamond_free[V](g: SimpleGraph[V]) {
    is_free_of(g, diamond4_graph) = forall(s: FiniteSet[V]) { is_diamond_free(g, s) }
} by {
    if is_free_of(g, diamond4_graph) {
        forall(s: FiniteSet[V]) {
            is_diamond_free_of_free_of_diamond(g, s)
            is_diamond_free(g, s)
        }
    }
    if forall(s: FiniteSet[V]) { is_diamond_free(g, s) } {
        free_of_diamond_of_is_diamond_free(g)
        is_free_of(g, diamond4_graph)
    }
    is_free_of(g, diamond4_graph) = forall(s: FiniteSet[V]) { is_diamond_free(g, s) }
}
