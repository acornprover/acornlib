/// Chordal graphs and perfect elimination orderings.

from nat import Nat, lt_suc, lt_trans, lte_trans, lte_and_lt, only_zero_lte_zero, lt_suc_right,
    not_lt_zero, zero_or_suc, lt_or_lte, small_mod, div_imp_mod, divides_self
from list import List, cons_unique_of_tail_unique_not_contains, singleton_contains_imp_eq,
    list_contains_implies_count_geq_one
from finite_set import FiniteSet
from data.finite.finite_set_filter import finite_set_filter, finite_set_filter_contains_eq
from data.basic.logic import false_implies
from data.list.list_cons_membership import cons_contains_eq, cons_contains_head,
    cons_contains_of_tail_contains, cons_contains_cases, nil_not_contains
from data.list.list_unique_cons import unique_cons_head_fresh, unique_cons_head_not_in_tail,
    unique_cons_parts
from graph.simple_graph import SimpleGraph, is_clique, complete_graph, complete_graph_adj_iff_ne
from graph.simple_graph_walks import simple_graph_walk, simple_graph_closed_walk,
    simple_graph_closed_walk_is_walk, simple_graph_walk_vertices, simple_graph_walk_cons_iff,
    simple_graph_walk_nil
from graph.simple_graph_tree import simple_graph_cycle, is_tree, tree_acyclic, is_acyclic
from graph.simple_graph_degree import neighborhood
from graph.simple_graph_cycle import cycle_graph, cycle_graph_adj_iff, cycle_graph_adj_suc,
    cycle_graph_adj_wrap
from graph.simple_graph_walk_edge import is_walk_edge, is_walk_edge_nil, is_walk_edge_cons,
    is_walk_edge_dir1, is_walk_edge_dir2, is_walk_edge_false_of_second_absent,
    is_walk_edge_false_of_first_absent
from graph.simple_graph_four_vertex import four_vertices, four_labels_lt, four_labels_ordered,
    four_labels_distinct, four_vertices_contains

numerals Nat

/// A chord of the cycle `(start, steps)`: an edge between two distinct vertices of the cycle
/// that are not consecutive along the cycle.
///
/// Two vertices of a simple cycle are consecutive exactly when the walk traverses the edge
/// between them in one direction or the other, so a chord is an adjacency that the cycle does
/// not traverse.
define has_chord[V](g: SimpleGraph[V], start: V, steps: List[V]) -> Bool {
    exists(x: V, y: V) {
        simple_graph_walk_vertices(start, steps).contains(x) and
        simple_graph_walk_vertices(start, steps).contains(y) and
        x != y and
        g.adj(x, y) and
        not is_walk_edge(start, steps, x, y) and
        not is_walk_edge(start, steps, y, x)
    }
}

/// An explicit non-consecutive adjacent pair of cycle vertices is a chord.
theorem has_chord_intro[V](g: SimpleGraph[V], start: V, steps: List[V], x: V, y: V) {
    simple_graph_walk_vertices(start, steps).contains(x) and
    simple_graph_walk_vertices(start, steps).contains(y) and
    x != y and g.adj(x, y) and
    not is_walk_edge(start, steps, x, y) and
    not is_walk_edge(start, steps, y, x) implies has_chord(g, start, steps)
} by {
    if simple_graph_walk_vertices(start, steps).contains(x) and
        simple_graph_walk_vertices(start, steps).contains(y) and
        x != y and g.adj(x, y) and
        not is_walk_edge(start, steps, x, y) and
        not is_walk_edge(start, steps, y, x) {
        has_chord(g, start, steps) = exists(a: V, b: V) {
            simple_graph_walk_vertices(start, steps).contains(a) and
            simple_graph_walk_vertices(start, steps).contains(b) and
            a != b and g.adj(a, b) and
            not is_walk_edge(start, steps, a, b) and
            not is_walk_edge(start, steps, b, a)
        }
        exists(a: V, b: V) {
            simple_graph_walk_vertices(start, steps).contains(a) and
            simple_graph_walk_vertices(start, steps).contains(b) and
            a != b and g.adj(a, b) and
            not is_walk_edge(start, steps, a, b) and
            not is_walk_edge(start, steps, b, a)
        }
    }
}

/// The graph is chordal on the vertex set `s`: every cycle of length at least four whose
/// vertices all lie in `s` has a chord.
define is_chordal[V](g: SimpleGraph[V], s: FiniteSet[V]) -> Bool {
    forall(start: V, steps: List[V]) {
        simple_graph_cycle(g, start, steps) and
        Nat.4 <= steps.length and
        (forall(x: V) { simple_graph_walk_vertices(start, steps).contains(x) implies s.contains(x) }) implies has_chord(g, start, steps)
    }
}

/// A cycle of length at least four inside the vertex set has a chord.
theorem is_chordal_apply[V](g: SimpleGraph[V], s: FiniteSet[V], start: V, steps: List[V]) {
    is_chordal(g, s) and simple_graph_cycle(g, start, steps) and Nat.4 <= steps.length and
        (forall(x: V) { simple_graph_walk_vertices(start, steps).contains(x) implies s.contains(x) }) implies has_chord(g, start, steps)
} by {
    if is_chordal(g, s) and simple_graph_cycle(g, start, steps) and Nat.4 <= steps.length and
        (forall(x: V) { simple_graph_walk_vertices(start, steps).contains(x) implies s.contains(x) }) {
        is_chordal(g, s) = forall(a: V, b: List[V]) {
            simple_graph_cycle(g, a, b) and
            Nat.4 <= b.length and
            (forall(x: V) { simple_graph_walk_vertices(a, b).contains(x) implies s.contains(x) }) implies has_chord(g, a, b)
        }
        forall(a: V, b: List[V]) {
            simple_graph_cycle(g, a, b) and
            Nat.4 <= b.length and
            (forall(x: V) { simple_graph_walk_vertices(a, b).contains(x) implies s.contains(x) }) implies has_chord(g, a, b)
        }
        simple_graph_cycle(g, start, steps) and Nat.4 <= steps.length and
            (forall(x: V) { simple_graph_walk_vertices(start, steps).contains(x) implies s.contains(x) }) implies has_chord(g, start, steps)
        has_chord(g, start, steps)
    }
}

/// A pointwise chord condition is chordality.
theorem is_chordal_intro[V](g: SimpleGraph[V], s: FiniteSet[V]) {
    (forall(start: V, steps: List[V]) {
        simple_graph_cycle(g, start, steps) and Nat.4 <= steps.length and
        (forall(x: V) { simple_graph_walk_vertices(start, steps).contains(x) implies s.contains(x) }) implies has_chord(g, start, steps)
    }) implies is_chordal(g, s)
} by {
    if forall(start: V, steps: List[V]) {
        simple_graph_cycle(g, start, steps) and Nat.4 <= steps.length and
        (forall(x: V) { simple_graph_walk_vertices(start, steps).contains(x) implies s.contains(x) }) implies has_chord(g, start, steps)
    } {
        is_chordal(g, s) = forall(a: V, b: List[V]) {
            simple_graph_cycle(g, a, b) and
            Nat.4 <= b.length and
            (forall(x: V) { simple_graph_walk_vertices(a, b).contains(x) implies s.contains(x) }) implies has_chord(g, a, b)
        }
        is_chordal(g, s)
    }
}

/// Membership in the list `items`, packaged as a predicate.
define list_membership_pred[V](items: List[V]) -> (V -> Bool) {
    function(x: V) {
        items.contains(x)
    }
}

/// The later neighbours of `v`: the neighbours of `v` in `s` that occur after `v` in the
/// ordering `rest`.
define later_neighbors[V](g: SimpleGraph[V], s: FiniteSet[V], v: V, rest: List[V]) -> FiniteSet[V] {
    finite_set_filter(neighborhood(g, s, v), list_membership_pred(rest))
}

/// A vertex lies among the later neighbours of `v` exactly when it is a neighbour of `v`
/// occurring later in the ordering.
theorem later_neighbors_contains_eq[V](g: SimpleGraph[V], s: FiniteSet[V], v: V, rest: List[V], w: V) {
    later_neighbors(g, s, v, rest).contains(w) =
        (s.contains(w) and g.adj(v, w) and rest.contains(w))
} by {
    finite_set_filter_contains_eq(neighborhood(g, s, v), list_membership_pred(rest), w)
    list_membership_pred(rest)(w) = rest.contains(w)
    later_neighbors(g, s, v, rest).contains(w) =
        (neighborhood(g, s, v).contains(w) and rest.contains(w))
    neighborhood(g, s, v).contains(w) = (s.contains(w) and g.adj(v, w))
}

/// True when the list `o` is a perfect elimination order of the vertices it lists: the later
/// neighbours of every vertex of `o` form a clique.
define is_peo_list[V](g: SimpleGraph[V], s: FiniteSet[V], o: List[V]) -> Bool {
    match o {
        List.nil[V] {
            true
        }
        List.cons(v, rest) {
            is_clique(g, later_neighbors(g, s, v, rest).underlying_set) and is_peo_list(g, s, rest)
        }
    }
}

/// A perfect elimination ordering of `(g, s)`: a list that lists the vertices of `s` without
/// repetition or omission, and whose later neighbours of every vertex form a clique.
///
/// By Fulkerson–Gross such an ordering exists exactly for chordal graphs.
define is_peo[V](g: SimpleGraph[V], s: FiniteSet[V], o: List[V]) -> Bool {
    (forall(v: V) { o.contains(v) implies s.contains(v) }) and
    (forall(v: V) { s.contains(v) implies o.contains(v) }) and
    o.is_unique and
    is_peo_list(g, s, o)
}

/// A list of length at least four is a cons of a cons of a cons of a cons.
theorem list_four_of_length[T](steps: List[T]) {
    Nat.4 <= steps.length implies exists(c1: T, c2: T, c3: T, rest: List[T]) {
        steps = List.cons(c1, List.cons(c2, List.cons(c3, rest)))
    }
} by {
    if Nat.4 <= steps.length {
        match steps {
            List.nil {
                List.nil[T].length = Nat.0
                Nat.4 <= List.nil[T].length
                Nat.4 <= Nat.0
                only_zero_lte_zero(Nat.4)
                Nat.4 = Nat.0
                false
            }
            List.cons(w1, rest1) {
                List.cons(w1, rest1).length = rest1.length + Nat.1
                match rest1 {
                    List.nil {
                        List.nil[T].length = Nat.0
                        Nat.4 <= List.cons(w1, List.nil[T]).length
                        Nat.4 <= Nat.0 + Nat.1
                        Nat.4 <= Nat.1
                        lt_suc(Nat.1)
                        Nat.1 < Nat.2
                        lt_suc(Nat.2)
                        Nat.2 < Nat.3
                        lt_trans(Nat.1, Nat.2, Nat.3)
                        Nat.1 < Nat.3
                        lt_suc(Nat.3)
                        Nat.3 < Nat.4
                        lt_trans(Nat.1, Nat.3, Nat.4)
                        Nat.1 < Nat.4
                        lte_and_lt(Nat.4, Nat.1, Nat.4)
                        Nat.4 < Nat.4
                        false
                    }
                    List.cons(w2, rest2) {
                        List.cons(w2, rest2).length = rest2.length + Nat.1
                        match rest2 {
                            List.nil {
                                List.nil[T].length = Nat.0
                                Nat.4 <= List.cons(w1, List.cons(w2, List.nil[T])).length
                                Nat.4 <= Nat.0 + Nat.1 + Nat.1
                                Nat.4 <= Nat.2
                                lt_suc(Nat.2)
                                Nat.2 < Nat.3
                                lt_suc(Nat.3)
                                Nat.3 < Nat.4
                                lt_trans(Nat.2, Nat.3, Nat.4)
                                Nat.2 < Nat.4
                                lte_and_lt(Nat.4, Nat.2, Nat.4)
                                Nat.4 < Nat.4
                                false
                            }
                            List.cons(w3, rest3) {
                                List.cons(w3, rest3).length = rest3.length + Nat.1
                                match rest3 {
                                    List.nil {
                                        List.nil[T].length = Nat.0
                                        Nat.4 <= List.cons(w1, List.cons(w2, List.cons(w3, List.nil[T]))).length
                                        Nat.4 <= Nat.0 + Nat.1 + Nat.1 + Nat.1
                                        Nat.4 <= Nat.3
                                        lt_suc(Nat.3)
                                        Nat.3 < Nat.4
                                        lte_and_lt(Nat.4, Nat.3, Nat.4)
                                        Nat.4 < Nat.4
                                        false
                                    }
                                    List.cons(w4, rest4) {
                                        steps = List.cons(w1, rest1)
                                        List.cons(w1, rest1) = List.cons(w1, List.cons(w2, rest2))
                                        List.cons(w1, List.cons(w2, rest2)) =
                                            List.cons(w1, List.cons(w2, List.cons(w3, rest3)))
                                        List.cons(w1, List.cons(w2, List.cons(w3, rest3))) =
                                            List.cons(w1, List.cons(w2, List.cons(w3, List.cons(w4, rest4))))
                                        steps = List.cons(w1, List.cons(w2, List.cons(w3, List.cons(w4, rest4))))
                                        exists(c1: T, c2: T, c3: T, rest: List[T]) {
                                            steps = List.cons(c1, List.cons(c2, List.cons(c3, rest)))
                                        }
                                    }
                                }
                            }
                        }
                    }
                }
            }
        }
    }
}

/// A list of length zero is the empty list.
theorem list_length_zero_imp_nil[T](items: List[T]) {
    items.length = Nat.0 implies items = List.nil[T]
} by {
    match items {
        List.nil {
            if items.length = Nat.0 {
                items = List.nil[T]
            }
        }
        List.cons(head, tail) {
            if items.length = Nat.0 {
                List.cons(head, tail).length = tail.length + Nat.1
                tail.length + Nat.1 = Nat.0
                tail.length < tail.length + Nat.1
                false
            }
        }
    }
}

/// A walk either is trivial or ends at its finish: the finish occurs among the targets.
///
/// Stated with the trivial case explicit so that the induction base needs no excluded-middle
/// reasoning: the empty target list is the empty walk, and every nonempty walk steps onto its
/// finish.
theorem simple_graph_walk_contains_finish[V](g: SimpleGraph[V], start: V, finish: V, steps: List[V]) {
    simple_graph_walk(g, start, finish, steps) implies steps = List.nil[V] or steps.contains(finish)
} by {
    define p(xs: List[V]) -> Bool {
        forall(s: V) {
            simple_graph_walk(g, s, finish, xs) implies (xs = List.nil[V] or xs.contains(finish))
        }
    }

    forall(s: V) {
        if simple_graph_walk(g, s, finish, List.nil[V]) {
            List.nil[V] = List.nil[V] or List.nil[V].contains(finish)
        }
        (simple_graph_walk(g, s, finish, List.nil[V]) implies (List.nil[V] = List.nil[V] or List.nil[V].contains(finish)))
    }
    p(List.nil[V])

    forall(next: V, tail: List[V]) {
        if p(tail) {
            forall(s: V) {
                if simple_graph_walk(g, s, finish, List.cons(next, tail)) {
                    simple_graph_walk_cons_iff(g, s, finish, next, tail)
                    simple_graph_walk(g, s, finish, List.cons(next, tail)) =
                        (g.adj(s, next) and simple_graph_walk(g, next, finish, tail))
                    simple_graph_walk(g, next, finish, tail)
                    p(tail)
                    p(tail) = forall(u: V) {
                        simple_graph_walk(g, u, finish, tail) implies (tail = List.nil[V] or tail.contains(finish))
                    }
                    simple_graph_walk(g, next, finish, tail) implies (tail = List.nil[V] or tail.contains(finish))
                    tail = List.nil[V] or tail.contains(finish)
                    if tail = List.nil[V] {
                        simple_graph_walk_nil(g, next, finish)
                        simple_graph_walk(g, next, finish, List.nil[V]) = (next = finish)
                        next = finish
                        cons_contains_head(next, List.nil[V])
                        List.cons(next, List.nil[V]).contains(next)
                        List.cons(next, tail).contains(finish)
                        List.cons(next, tail) = List.nil[V] or List.cons(next, tail).contains(finish)
                    } else {
                        if tail = List.nil[V] or tail.contains(finish) {
                            if tail = List.nil[V] {
                                false
                            }
                            if tail != List.nil[V] {
                                tail.contains(finish)
                            }
                            tail.contains(finish)
                        }
                        tail.contains(finish)
                        cons_contains_of_tail_contains(next, tail, finish)
                        List.cons(next, tail).contains(finish)
                        List.cons(next, tail) = List.nil[V] or List.cons(next, tail).contains(finish)
                    }
                }
                (simple_graph_walk(g, s, finish, List.cons(next, tail)) implies (List.cons(next, tail) = List.nil[V] or List.cons(next, tail).contains(finish)))
            }
            p(List.cons(next, tail))
        }
    }
    forall(next: V, tail: List[V]) { p(tail) implies p(List.cons(next, tail)) }
    function[T0](x0: List[T0] -> Bool) {
        not forall(x1: T0, x2: List[T0]) { not x0(x2) or x0(List.cons(x1, x2)) } or
        not x0(List.nil[T0]) or forall(x3: List[T0]) { x0(x3) } = true
    }[V](p)
    function[T0](x0: List[T0] -> Bool, x1: List[T0]) {
        not forall(x2: T0, x3: List[T0]) { not x0(x3) or x0(List.cons(x2, x3)) } or
        not x0(List.nil[T0]) or x0(x1)
    }[V](p, steps)
    p(steps)
    if simple_graph_walk(g, start, finish, steps) {
        p(steps) = forall(u: V) {
            simple_graph_walk(g, u, finish, steps) implies (steps = List.nil[V] or steps.contains(finish))
        }
        forall(u: V) {
            simple_graph_walk(g, u, finish, steps) implies (steps = List.nil[V] or steps.contains(finish))
        }
        steps = List.nil[V] or steps.contains(finish)
    }
}

/// The canonical four-cycle: the edge targets `1, 2, 3, 0` of the closed walk from `0`.
let cycle4_steps: List[Nat] = List.cons(Nat.1, List.cons(Nat.2, List.cons(Nat.3, List.cons(Nat.0, List.nil[Nat]))))

/// The residues of the labels below four are the labels themselves.
theorem cycle4_mod_one {
    Nat.1.mod(Nat.4) = Nat.1
} by {
    four_labels_lt
    (Nat.0 < Nat.4 and Nat.1 < Nat.4 and Nat.2 < Nat.4 and Nat.3 < Nat.4)
    Nat.1 < Nat.4
    small_mod(Nat.1, Nat.4)
}

/// The residue of `2` modulo four.
theorem cycle4_mod_two {
    Nat.2.mod(Nat.4) = Nat.2
} by {
    four_labels_lt
    (Nat.0 < Nat.4 and Nat.1 < Nat.4 and Nat.2 < Nat.4 and Nat.3 < Nat.4)
    Nat.2 < Nat.4
    small_mod(Nat.2, Nat.4)
}

/// The residue of `3` modulo four.
theorem cycle4_mod_three {
    Nat.3.mod(Nat.4) = Nat.3
} by {
    four_labels_lt
    (Nat.0 < Nat.4 and Nat.1 < Nat.4 and Nat.2 < Nat.4 and Nat.3 < Nat.4)
    Nat.3 < Nat.4
    small_mod(Nat.3, Nat.4)
}

/// The residue of `4` modulo four is `0`: four is its own multiple.
theorem cycle4_mod_four {
    Nat.4.mod(Nat.4) = Nat.0
} by {
    divides_self(Nat.4)
    div_imp_mod(Nat.4, Nat.4)
}

/// The residues `1` and `3` modulo four are not `0` or `2`.
theorem cycle4_mod_one_ne {
    not (Nat.1.mod(Nat.4) = Nat.0) and not (Nat.1.mod(Nat.4) = Nat.2)
} by {
    cycle4_mod_one
    Nat.1.mod(Nat.4) = Nat.1
    if Nat.1.mod(Nat.4) = Nat.0 {
        Nat.1 = Nat.0
        false
    }
    not (Nat.1.mod(Nat.4) = Nat.0)
    if Nat.1.mod(Nat.4) = Nat.2 {
        Nat.1 = Nat.2
        false
    }
    not (Nat.1.mod(Nat.4) = Nat.2)
}

/// The residues `2` and `4` modulo four are not `1` or `3`.
theorem cycle4_mod_two_ne {
    not (Nat.2.mod(Nat.4) = Nat.1) and not (Nat.2.mod(Nat.4) = Nat.3)
} by {
    cycle4_mod_two
    Nat.2.mod(Nat.4) = Nat.2
    if Nat.2.mod(Nat.4) = Nat.1 {
        Nat.2 = Nat.1
        false
    }
    not (Nat.2.mod(Nat.4) = Nat.1)
    if Nat.2.mod(Nat.4) = Nat.3 {
        Nat.2 = Nat.3
        false
    }
    not (Nat.2.mod(Nat.4) = Nat.3)
}

/// The residues `3` and `1` modulo four are not `0` or `2`.
theorem cycle4_mod_three_ne {
    not (Nat.3.mod(Nat.4) = Nat.0) and not (Nat.3.mod(Nat.4) = Nat.2)
} by {
    cycle4_mod_three
    Nat.3.mod(Nat.4) = Nat.3
    if Nat.3.mod(Nat.4) = Nat.0 {
        Nat.3 = Nat.0
        false
    }
    not (Nat.3.mod(Nat.4) = Nat.0)
    if Nat.3.mod(Nat.4) = Nat.2 {
        Nat.3 = Nat.2
        false
    }
    not (Nat.3.mod(Nat.4) = Nat.2)
}

/// The residues `4` and `2` modulo four are not `1` or `3`.
theorem cycle4_mod_four_ne {
    not (Nat.4.mod(Nat.4) = Nat.1) and not (Nat.4.mod(Nat.4) = Nat.3)
} by {
    cycle4_mod_four
    Nat.4.mod(Nat.4) = Nat.0
    if Nat.4.mod(Nat.4) = Nat.1 {
        Nat.0 = Nat.1
        false
    }
    not (Nat.4.mod(Nat.4) = Nat.1)
    if Nat.4.mod(Nat.4) = Nat.3 {
        Nat.0 = Nat.3
        false
    }
    not (Nat.4.mod(Nat.4) = Nat.3)
}

/// The canonical four-cycle traverses the edge `0`--`1`.
theorem cycle4_walk_edge_01 {
    is_walk_edge(Nat.0, cycle4_steps, Nat.0, Nat.1)
} by {
    is_walk_edge_cons(Nat.0, Nat.1, List.cons(Nat.2, List.cons(Nat.3, List.cons(Nat.0, List.nil[Nat]))), Nat.0, Nat.1)
    is_walk_edge(Nat.0, List.cons(Nat.1, List.cons(Nat.2, List.cons(Nat.3, List.cons(Nat.0, List.nil[Nat])))), Nat.0, Nat.1) =
        ((Nat.0 = Nat.0 and Nat.1 = Nat.1) or is_walk_edge(Nat.1, List.cons(Nat.2, List.cons(Nat.3, List.cons(Nat.0, List.nil[Nat]))), Nat.0, Nat.1))
    Nat.0 = Nat.0 and Nat.1 = Nat.1
    is_walk_edge(Nat.0, cycle4_steps, Nat.0, Nat.1)
}

/// The canonical four-cycle traverses the edge `1`--`2`.
theorem cycle4_walk_edge_12 {
    is_walk_edge(Nat.0, cycle4_steps, Nat.1, Nat.2)
} by {
    is_walk_edge_cons(Nat.0, Nat.1, List.cons(Nat.2, List.cons(Nat.3, List.cons(Nat.0, List.nil[Nat]))), Nat.1, Nat.2)
    is_walk_edge(Nat.0, List.cons(Nat.1, List.cons(Nat.2, List.cons(Nat.3, List.cons(Nat.0, List.nil[Nat])))), Nat.1, Nat.2) =
        ((Nat.0 = Nat.1 and Nat.1 = Nat.2) or is_walk_edge(Nat.1, List.cons(Nat.2, List.cons(Nat.3, List.cons(Nat.0, List.nil[Nat]))), Nat.1, Nat.2))
    not (Nat.0 = Nat.1 and Nat.1 = Nat.2)
    is_walk_edge_cons(Nat.1, Nat.2, List.cons(Nat.3, List.cons(Nat.0, List.nil[Nat])), Nat.1, Nat.2)
    is_walk_edge(Nat.1, List.cons(Nat.2, List.cons(Nat.3, List.cons(Nat.0, List.nil[Nat]))), Nat.1, Nat.2) =
        ((Nat.1 = Nat.1 and Nat.2 = Nat.2) or is_walk_edge(Nat.2, List.cons(Nat.3, List.cons(Nat.0, List.nil[Nat])), Nat.1, Nat.2))
    Nat.1 = Nat.1 and Nat.2 = Nat.2
    is_walk_edge(Nat.0, cycle4_steps, Nat.1, Nat.2)
}

/// The canonical four-cycle traverses the edge `2`--`3`.
theorem cycle4_walk_edge_23 {
    is_walk_edge(Nat.0, cycle4_steps, Nat.2, Nat.3)
} by {
    is_walk_edge_cons(Nat.0, Nat.1, List.cons(Nat.2, List.cons(Nat.3, List.cons(Nat.0, List.nil[Nat]))), Nat.2, Nat.3)
    is_walk_edge(Nat.0, List.cons(Nat.1, List.cons(Nat.2, List.cons(Nat.3, List.cons(Nat.0, List.nil[Nat])))), Nat.2, Nat.3) =
        ((Nat.0 = Nat.2 and Nat.1 = Nat.3) or is_walk_edge(Nat.1, List.cons(Nat.2, List.cons(Nat.3, List.cons(Nat.0, List.nil[Nat]))), Nat.2, Nat.3))
    not (Nat.0 = Nat.2 and Nat.1 = Nat.3)
    is_walk_edge_cons(Nat.1, Nat.2, List.cons(Nat.3, List.cons(Nat.0, List.nil[Nat])), Nat.2, Nat.3)
    is_walk_edge(Nat.1, List.cons(Nat.2, List.cons(Nat.3, List.cons(Nat.0, List.nil[Nat]))), Nat.2, Nat.3) =
        ((Nat.1 = Nat.2 and Nat.2 = Nat.3) or is_walk_edge(Nat.2, List.cons(Nat.3, List.cons(Nat.0, List.nil[Nat])), Nat.2, Nat.3))
    not (Nat.1 = Nat.2 and Nat.2 = Nat.3)
    is_walk_edge_cons(Nat.2, Nat.3, List.cons(Nat.0, List.nil[Nat]), Nat.2, Nat.3)
    is_walk_edge(Nat.2, List.cons(Nat.3, List.cons(Nat.0, List.nil[Nat])), Nat.2, Nat.3) =
        ((Nat.2 = Nat.2 and Nat.3 = Nat.3) or is_walk_edge(Nat.3, List.cons(Nat.0, List.nil[Nat]), Nat.2, Nat.3))
    Nat.2 = Nat.2 and Nat.3 = Nat.3
    is_walk_edge(Nat.0, cycle4_steps, Nat.2, Nat.3)
}

/// The canonical four-cycle traverses the edge `3`--`0`.
theorem cycle4_walk_edge_30 {
    is_walk_edge(Nat.0, cycle4_steps, Nat.3, Nat.0)
} by {
    is_walk_edge_cons(Nat.0, Nat.1, List.cons(Nat.2, List.cons(Nat.3, List.cons(Nat.0, List.nil[Nat]))), Nat.3, Nat.0)
    is_walk_edge(Nat.0, List.cons(Nat.1, List.cons(Nat.2, List.cons(Nat.3, List.cons(Nat.0, List.nil[Nat])))), Nat.3, Nat.0) =
        ((Nat.0 = Nat.3 and Nat.1 = Nat.0) or is_walk_edge(Nat.1, List.cons(Nat.2, List.cons(Nat.3, List.cons(Nat.0, List.nil[Nat]))), Nat.3, Nat.0))
    not (Nat.0 = Nat.3 and Nat.1 = Nat.0)
    is_walk_edge_cons(Nat.1, Nat.2, List.cons(Nat.3, List.cons(Nat.0, List.nil[Nat])), Nat.3, Nat.0)
    is_walk_edge(Nat.1, List.cons(Nat.2, List.cons(Nat.3, List.cons(Nat.0, List.nil[Nat]))), Nat.3, Nat.0) =
        ((Nat.1 = Nat.3 and Nat.2 = Nat.0) or is_walk_edge(Nat.2, List.cons(Nat.3, List.cons(Nat.0, List.nil[Nat])), Nat.3, Nat.0))
    not (Nat.1 = Nat.3 and Nat.2 = Nat.0)
    is_walk_edge_cons(Nat.2, Nat.3, List.cons(Nat.0, List.nil[Nat]), Nat.3, Nat.0)
    is_walk_edge(Nat.2, List.cons(Nat.3, List.cons(Nat.0, List.nil[Nat])), Nat.3, Nat.0) =
        ((Nat.2 = Nat.3 and Nat.3 = Nat.0) or is_walk_edge(Nat.3, List.cons(Nat.0, List.nil[Nat]), Nat.3, Nat.0))
    not (Nat.2 = Nat.3 and Nat.3 = Nat.0)
    is_walk_edge_cons(Nat.3, Nat.0, List.nil[Nat], Nat.3, Nat.0)
    is_walk_edge(Nat.3, List.cons(Nat.0, List.nil[Nat]), Nat.3, Nat.0) =
        ((Nat.3 = Nat.3 and Nat.0 = Nat.0) or is_walk_edge(Nat.0, List.nil[Nat], Nat.3, Nat.0))
    Nat.3 = Nat.3 and Nat.0 = Nat.0
    is_walk_edge(Nat.0, cycle4_steps, Nat.3, Nat.0)
}

/// The pair `0`--`2` is a diagonal of the four-cycle: not an edge.
theorem cycle4_diag_02 {
    not cycle_graph(Nat.4).adj(Nat.0, Nat.2)
} by {
    cycle_graph_adj_iff(Nat.4, Nat.0, Nat.2)
    (cycle_graph(Nat.4).adj(Nat.0, Nat.2) =
        (Nat.0 != Nat.2 and (Nat.0.suc.mod(Nat.4) = Nat.2 or Nat.2.suc.mod(Nat.4) = Nat.0)))
    if cycle_graph(Nat.4).adj(Nat.0, Nat.2) {
        Nat.0.suc.mod(Nat.4) = Nat.2 or Nat.2.suc.mod(Nat.4) = Nat.0
        cycle4_mod_one_ne
        not (Nat.1.mod(Nat.4) = Nat.2)
        cycle4_mod_three_ne
        not (Nat.3.mod(Nat.4) = Nat.0)
        if Nat.0.suc.mod(Nat.4) = Nat.2 {
            Nat.1.mod(Nat.4) = Nat.2
            false
        }
        if not (Nat.0.suc.mod(Nat.4) = Nat.2) {
            Nat.2.suc.mod(Nat.4) = Nat.0
            Nat.3.mod(Nat.4) = Nat.0
            false
        }
        false
    }
    not cycle_graph(Nat.4).adj(Nat.0, Nat.2)
}

/// The pair `1`--`3` is a diagonal of the four-cycle: not an edge.
theorem cycle4_diag_13 {
    not cycle_graph(Nat.4).adj(Nat.1, Nat.3)
} by {
    cycle_graph_adj_iff(Nat.4, Nat.1, Nat.3)
    (cycle_graph(Nat.4).adj(Nat.1, Nat.3) =
        (Nat.1 != Nat.3 and (Nat.1.suc.mod(Nat.4) = Nat.3 or Nat.3.suc.mod(Nat.4) = Nat.1)))
    if cycle_graph(Nat.4).adj(Nat.1, Nat.3) {
        Nat.1.suc.mod(Nat.4) = Nat.3 or Nat.3.suc.mod(Nat.4) = Nat.1
        cycle4_mod_two_ne
        not (Nat.2.mod(Nat.4) = Nat.3)
        cycle4_mod_four_ne
        not (Nat.4.mod(Nat.4) = Nat.1)
        if Nat.1.suc.mod(Nat.4) = Nat.3 {
            Nat.2.mod(Nat.4) = Nat.3
            false
        }
        if not (Nat.1.suc.mod(Nat.4) = Nat.3) {
            Nat.3.suc.mod(Nat.4) = Nat.1
            Nat.4.mod(Nat.4) = Nat.1
            false
        }
        false
    }
    not cycle_graph(Nat.4).adj(Nat.1, Nat.3)
}

/// The pair `2`--`0` is a diagonal of the four-cycle: not an edge.
theorem cycle4_diag_20 {
    not cycle_graph(Nat.4).adj(Nat.2, Nat.0)
} by {
    cycle_graph_adj_iff(Nat.4, Nat.2, Nat.0)
    (cycle_graph(Nat.4).adj(Nat.2, Nat.0) =
        (Nat.2 != Nat.0 and (Nat.2.suc.mod(Nat.4) = Nat.0 or Nat.0.suc.mod(Nat.4) = Nat.2)))
    if cycle_graph(Nat.4).adj(Nat.2, Nat.0) {
        Nat.2.suc.mod(Nat.4) = Nat.0 or Nat.0.suc.mod(Nat.4) = Nat.2
        cycle4_mod_three_ne
        not (Nat.3.mod(Nat.4) = Nat.0)
        cycle4_mod_one_ne
        not (Nat.1.mod(Nat.4) = Nat.2)
        if Nat.2.suc.mod(Nat.4) = Nat.0 {
            Nat.3.mod(Nat.4) = Nat.0
            false
        }
        if not (Nat.2.suc.mod(Nat.4) = Nat.0) {
            Nat.0.suc.mod(Nat.4) = Nat.2
            Nat.1.mod(Nat.4) = Nat.2
            false
        }
        false
    }
    not cycle_graph(Nat.4).adj(Nat.2, Nat.0)
}

/// The pair `3`--`1` is a diagonal of the four-cycle: not an edge.
theorem cycle4_diag_31 {
    not cycle_graph(Nat.4).adj(Nat.3, Nat.1)
} by {
    cycle_graph_adj_iff(Nat.4, Nat.3, Nat.1)
    (cycle_graph(Nat.4).adj(Nat.3, Nat.1) =
        (Nat.3 != Nat.1 and (Nat.3.suc.mod(Nat.4) = Nat.1 or Nat.1.suc.mod(Nat.4) = Nat.3)))
    if cycle_graph(Nat.4).adj(Nat.3, Nat.1) {
        Nat.3.suc.mod(Nat.4) = Nat.1 or Nat.1.suc.mod(Nat.4) = Nat.3
        cycle4_mod_four_ne
        not (Nat.4.mod(Nat.4) = Nat.1)
        cycle4_mod_two_ne
        not (Nat.2.mod(Nat.4) = Nat.3)
        if Nat.3.suc.mod(Nat.4) = Nat.1 {
            Nat.4.mod(Nat.4) = Nat.1
            false
        }
        if not (Nat.3.suc.mod(Nat.4) = Nat.1) {
            Nat.1.suc.mod(Nat.4) = Nat.3
            Nat.2.mod(Nat.4) = Nat.3
            false
        }
        false
    }
    not cycle_graph(Nat.4).adj(Nat.3, Nat.1)
}

/// An adjacency from `0` in the four-cycle is one of the two cycle edges at `0`.
theorem cycle4_adj_zero_imp_walk_edge(y: Nat) {
    (y = Nat.0 or y = Nat.1 or y = Nat.2 or y = Nat.3) and cycle_graph(Nat.4).adj(Nat.0, y) implies (is_walk_edge(Nat.0, cycle4_steps, Nat.0, y) or is_walk_edge(Nat.0, cycle4_steps, y, Nat.0))
} by {
    if (y = Nat.0 or y = Nat.1 or y = Nat.2 or y = Nat.3) and cycle_graph(Nat.4).adj(Nat.0, y) {
        cycle_graph_adj_iff(Nat.4, Nat.0, y)
        (cycle_graph(Nat.4).adj(Nat.0, y) = (Nat.0 != y and (Nat.0.suc.mod(Nat.4) = y or y.suc.mod(Nat.4) = Nat.0)))
        if y = Nat.0 {
            Nat.0 != y
            Nat.0 != Nat.0
            false
        }
        if y = Nat.1 {
            cycle4_walk_edge_01
            is_walk_edge(Nat.0, cycle4_steps, Nat.0, Nat.1)
            is_walk_edge(Nat.0, cycle4_steps, Nat.0, y) or is_walk_edge(Nat.0, cycle4_steps, y, Nat.0)
        }
        if y = Nat.2 {
            cycle4_diag_02
            not cycle_graph(Nat.4).adj(Nat.0, Nat.2)
            false
        }
        if y = Nat.3 {
            cycle4_walk_edge_30
            is_walk_edge(Nat.0, cycle4_steps, Nat.3, Nat.0)
            is_walk_edge(Nat.0, cycle4_steps, y, Nat.0)
            is_walk_edge(Nat.0, cycle4_steps, Nat.0, y) or is_walk_edge(Nat.0, cycle4_steps, y, Nat.0)
        }
        is_walk_edge(Nat.0, cycle4_steps, Nat.0, y) or is_walk_edge(Nat.0, cycle4_steps, y, Nat.0)
    }
}

/// An adjacency from `1` in the four-cycle is one of the two cycle edges at `1`.
theorem cycle4_adj_one_imp_walk_edge(y: Nat) {
    (y = Nat.0 or y = Nat.1 or y = Nat.2 or y = Nat.3) and cycle_graph(Nat.4).adj(Nat.1, y) implies (is_walk_edge(Nat.0, cycle4_steps, Nat.1, y) or is_walk_edge(Nat.0, cycle4_steps, y, Nat.1))
} by {
    if (y = Nat.0 or y = Nat.1 or y = Nat.2 or y = Nat.3) and cycle_graph(Nat.4).adj(Nat.1, y) {
        cycle_graph_adj_iff(Nat.4, Nat.1, y)
        (cycle_graph(Nat.4).adj(Nat.1, y) = (Nat.1 != y and (Nat.1.suc.mod(Nat.4) = y or y.suc.mod(Nat.4) = Nat.1)))
        if y = Nat.0 {
            cycle4_walk_edge_01
            is_walk_edge(Nat.0, cycle4_steps, Nat.0, Nat.1)
            is_walk_edge(Nat.0, cycle4_steps, y, Nat.1)
            is_walk_edge(Nat.0, cycle4_steps, Nat.1, y) or is_walk_edge(Nat.0, cycle4_steps, y, Nat.1)
        }
        if y = Nat.1 {
            Nat.1 != y
            Nat.1 != Nat.1
            false
        }
        if y = Nat.2 {
            cycle4_walk_edge_12
            is_walk_edge(Nat.0, cycle4_steps, Nat.1, Nat.2)
            is_walk_edge(Nat.0, cycle4_steps, Nat.1, y) or is_walk_edge(Nat.0, cycle4_steps, y, Nat.1)
        }
        if y = Nat.3 {
            cycle4_diag_13
            not cycle_graph(Nat.4).adj(Nat.1, Nat.3)
            false
        }
        is_walk_edge(Nat.0, cycle4_steps, Nat.1, y) or is_walk_edge(Nat.0, cycle4_steps, y, Nat.1)
    }
}

/// An adjacency from `2` in the four-cycle is one of the two cycle edges at `2`.
theorem cycle4_adj_two_imp_walk_edge(y: Nat) {
    (y = Nat.0 or y = Nat.1 or y = Nat.2 or y = Nat.3) and cycle_graph(Nat.4).adj(Nat.2, y) implies (is_walk_edge(Nat.0, cycle4_steps, Nat.2, y) or is_walk_edge(Nat.0, cycle4_steps, y, Nat.2))
} by {
    if (y = Nat.0 or y = Nat.1 or y = Nat.2 or y = Nat.3) and cycle_graph(Nat.4).adj(Nat.2, y) {
        cycle_graph_adj_iff(Nat.4, Nat.2, y)
        (cycle_graph(Nat.4).adj(Nat.2, y) = (Nat.2 != y and (Nat.2.suc.mod(Nat.4) = y or y.suc.mod(Nat.4) = Nat.2)))
        if y = Nat.0 {
            cycle4_diag_20
            not cycle_graph(Nat.4).adj(Nat.2, Nat.0)
            false
        }
        if y = Nat.1 {
            cycle4_walk_edge_12
            is_walk_edge(Nat.0, cycle4_steps, Nat.1, Nat.2)
            is_walk_edge(Nat.0, cycle4_steps, y, Nat.2)
            is_walk_edge(Nat.0, cycle4_steps, Nat.2, y) or is_walk_edge(Nat.0, cycle4_steps, y, Nat.2)
        }
        if y = Nat.2 {
            Nat.2 != y
            Nat.2 != Nat.2
            false
        }
        if y = Nat.3 {
            cycle4_walk_edge_23
            is_walk_edge(Nat.0, cycle4_steps, Nat.2, Nat.3)
            is_walk_edge(Nat.0, cycle4_steps, Nat.2, y) or is_walk_edge(Nat.0, cycle4_steps, y, Nat.2)
        }
        is_walk_edge(Nat.0, cycle4_steps, Nat.2, y) or is_walk_edge(Nat.0, cycle4_steps, y, Nat.2)
    }
}

/// An adjacency from `3` in the four-cycle is one of the two cycle edges at `3`.
theorem cycle4_adj_three_imp_walk_edge(y: Nat) {
    (y = Nat.0 or y = Nat.1 or y = Nat.2 or y = Nat.3) and cycle_graph(Nat.4).adj(Nat.3, y) implies (is_walk_edge(Nat.0, cycle4_steps, Nat.3, y) or is_walk_edge(Nat.0, cycle4_steps, y, Nat.3))
} by {
    if (y = Nat.0 or y = Nat.1 or y = Nat.2 or y = Nat.3) and cycle_graph(Nat.4).adj(Nat.3, y) {
        cycle_graph_adj_iff(Nat.4, Nat.3, y)
        (cycle_graph(Nat.4).adj(Nat.3, y) = (Nat.3 != y and (Nat.3.suc.mod(Nat.4) = y or y.suc.mod(Nat.4) = Nat.3)))
        if y = Nat.0 {
            cycle4_walk_edge_30
            is_walk_edge(Nat.0, cycle4_steps, Nat.3, Nat.0)
            is_walk_edge(Nat.0, cycle4_steps, Nat.3, y) or is_walk_edge(Nat.0, cycle4_steps, y, Nat.3)
        }
        if y = Nat.1 {
            cycle4_diag_31
            not cycle_graph(Nat.4).adj(Nat.3, Nat.1)
            false
        }
        if y = Nat.2 {
            cycle4_walk_edge_23
            is_walk_edge(Nat.0, cycle4_steps, Nat.2, Nat.3)
            is_walk_edge(Nat.0, cycle4_steps, y, Nat.3)
            is_walk_edge(Nat.0, cycle4_steps, Nat.3, y) or is_walk_edge(Nat.0, cycle4_steps, y, Nat.3)
        }
        if y = Nat.3 {
            Nat.3 != y
            Nat.3 != Nat.3
            false
        }
        is_walk_edge(Nat.0, cycle4_steps, Nat.3, y) or is_walk_edge(Nat.0, cycle4_steps, y, Nat.3)
    }
}

/// A list whose count of `x` is zero does not contain `x`.
theorem list_count_zero_imp_not_contains[T](items: List[T], x: T) {
    items.count(x) = Nat.0 implies not items.contains(x)
} by {
    if items.count(x) = Nat.0 {
        if items.contains(x) {
            list_contains_implies_count_geq_one(items, x)
            items.contains(x) implies items.count(x) >= Nat.1
            items.count(x) >= Nat.1
            items.count(x) = Nat.0
            items.count(x) <= Nat.0
            lte_trans(Nat.1, items.count(x), Nat.0)
            Nat.1 <= Nat.0
            lt_suc(Nat.0)
            Nat.0 < Nat.1
            lte_and_lt(Nat.1, Nat.0, Nat.1)
            Nat.1 < Nat.1
            false
        }
        not items.contains(x)
    }
}

/// The vertex `3` is not among the targets of the one-element list `[0]`.
theorem cycle4_not_contains_three_in_zero {
    not List.cons(Nat.0, List.nil[Nat]).contains(Nat.3)
} by {
    List.nil[Nat].count(Nat.3) = Nat.0
    List.cons(Nat.0, List.nil[Nat]).count(Nat.3) =
        (if Nat.0 = Nat.3 { Nat.1 + List.nil[Nat].count(Nat.3) } else { List.nil[Nat].count(Nat.3) })
    Nat.0 != Nat.3
    (if Nat.0 = Nat.3 { Nat.1 + List.nil[Nat].count(Nat.3) } else { List.nil[Nat].count(Nat.3) }) =
        List.nil[Nat].count(Nat.3)
    List.cons(Nat.0, List.nil[Nat]).count(Nat.3) = Nat.0
    list_count_zero_imp_not_contains(List.cons(Nat.0, List.nil[Nat]), Nat.3)
    not List.cons(Nat.0, List.nil[Nat]).contains(Nat.3)
}

/// The vertex `2` is not among the targets of the two-element list `[3, 0]`.
theorem cycle4_not_contains_two_in_three_zero {
    not List.cons(Nat.3, List.cons(Nat.0, List.nil[Nat])).contains(Nat.2)
} by {
    List.nil[Nat].count(Nat.2) = Nat.0
    List.cons(Nat.0, List.nil[Nat]).count(Nat.2) =
        (if Nat.0 = Nat.2 { Nat.1 + List.nil[Nat].count(Nat.2) } else { List.nil[Nat].count(Nat.2) })
    Nat.0 != Nat.2
    (if Nat.0 = Nat.2 { Nat.1 + List.nil[Nat].count(Nat.2) } else { List.nil[Nat].count(Nat.2) }) =
        List.nil[Nat].count(Nat.2)
    List.cons(Nat.0, List.nil[Nat]).count(Nat.2) = Nat.0
    List.cons(Nat.3, List.cons(Nat.0, List.nil[Nat])).count(Nat.2) =
        (if Nat.3 = Nat.2 { Nat.1 + List.cons(Nat.0, List.nil[Nat]).count(Nat.2) } else { List.cons(Nat.0, List.nil[Nat]).count(Nat.2) })
    Nat.3 != Nat.2
    (if Nat.3 = Nat.2 { Nat.1 + List.cons(Nat.0, List.nil[Nat]).count(Nat.2) } else { List.cons(Nat.0, List.nil[Nat]).count(Nat.2) }) =
        List.cons(Nat.0, List.nil[Nat]).count(Nat.2)
    List.cons(Nat.3, List.cons(Nat.0, List.nil[Nat])).count(Nat.2) = Nat.0
    list_count_zero_imp_not_contains(List.cons(Nat.3, List.cons(Nat.0, List.nil[Nat])), Nat.2)
    not List.cons(Nat.3, List.cons(Nat.0, List.nil[Nat])).contains(Nat.2)
}

/// The vertex `1` is not among the targets of the three-element list `[2, 3, 0]`.
theorem cycle4_not_contains_one_in_two_three_zero {
    not List.cons(Nat.2, List.cons(Nat.3, List.cons(Nat.0, List.nil[Nat]))).contains(Nat.1)
} by {
    List.nil[Nat].count(Nat.1) = Nat.0
    List.cons(Nat.0, List.nil[Nat]).count(Nat.1) =
        (if Nat.0 = Nat.1 { Nat.1 + List.nil[Nat].count(Nat.1) } else { List.nil[Nat].count(Nat.1) })
    Nat.0 != Nat.1
    (if Nat.0 = Nat.1 { Nat.1 + List.nil[Nat].count(Nat.1) } else { List.nil[Nat].count(Nat.1) }) =
        List.nil[Nat].count(Nat.1)
    List.cons(Nat.0, List.nil[Nat]).count(Nat.1) = Nat.0
    List.cons(Nat.3, List.cons(Nat.0, List.nil[Nat])).count(Nat.1) =
        (if Nat.3 = Nat.1 { Nat.1 + List.cons(Nat.0, List.nil[Nat]).count(Nat.1) } else { List.cons(Nat.0, List.nil[Nat]).count(Nat.1) })
    Nat.3 != Nat.1
    (if Nat.3 = Nat.1 { Nat.1 + List.cons(Nat.0, List.nil[Nat]).count(Nat.1) } else { List.cons(Nat.0, List.nil[Nat]).count(Nat.1) }) =
        List.cons(Nat.0, List.nil[Nat]).count(Nat.1)
    List.cons(Nat.3, List.cons(Nat.0, List.nil[Nat])).count(Nat.1) = Nat.0
    List.cons(Nat.2, List.cons(Nat.3, List.cons(Nat.0, List.nil[Nat]))).count(Nat.1) =
        (if Nat.2 = Nat.1 { Nat.1 + List.cons(Nat.3, List.cons(Nat.0, List.nil[Nat])).count(Nat.1) } else { List.cons(Nat.3, List.cons(Nat.0, List.nil[Nat])).count(Nat.1) })
    Nat.2 != Nat.1
    (if Nat.2 = Nat.1 { Nat.1 + List.cons(Nat.3, List.cons(Nat.0, List.nil[Nat])).count(Nat.1) } else { List.cons(Nat.3, List.cons(Nat.0, List.nil[Nat])).count(Nat.1) }) =
        List.cons(Nat.3, List.cons(Nat.0, List.nil[Nat])).count(Nat.1)
    List.cons(Nat.2, List.cons(Nat.3, List.cons(Nat.0, List.nil[Nat]))).count(Nat.1) = Nat.0
    list_count_zero_imp_not_contains(List.cons(Nat.2, List.cons(Nat.3, List.cons(Nat.0, List.nil[Nat]))), Nat.1)
    not List.cons(Nat.2, List.cons(Nat.3, List.cons(Nat.0, List.nil[Nat]))).contains(Nat.1)
}

/// The canonical four-cycle repeats no vertex: its target list is unique.
theorem cycle4_steps_unique {
    cycle4_steps.is_unique
} by {
    nil_not_contains(Nat.0)
    not List.nil[Nat].contains(Nat.0)
    List.nil[Nat].is_unique
    cons_unique_of_tail_unique_not_contains(Nat.0, List.nil[Nat])
    List.cons(Nat.0, List.nil[Nat]).is_unique
    cycle4_not_contains_three_in_zero
    not List.cons(Nat.0, List.nil[Nat]).contains(Nat.3)
    cons_unique_of_tail_unique_not_contains(Nat.3, List.cons(Nat.0, List.nil[Nat]))
    List.cons(Nat.3, List.cons(Nat.0, List.nil[Nat])).is_unique
    cycle4_not_contains_two_in_three_zero
    not List.cons(Nat.3, List.cons(Nat.0, List.nil[Nat])).contains(Nat.2)
    cons_unique_of_tail_unique_not_contains(Nat.2, List.cons(Nat.3, List.cons(Nat.0, List.nil[Nat])))
    List.cons(Nat.2, List.cons(Nat.3, List.cons(Nat.0, List.nil[Nat]))).is_unique
    cycle4_not_contains_one_in_two_three_zero
    not List.cons(Nat.2, List.cons(Nat.3, List.cons(Nat.0, List.nil[Nat]))).contains(Nat.1)
    cons_unique_of_tail_unique_not_contains(Nat.1, List.cons(Nat.2, List.cons(Nat.3, List.cons(Nat.0, List.nil[Nat]))))
    List.cons(Nat.1, List.cons(Nat.2, List.cons(Nat.3, List.cons(Nat.0, List.nil[Nat])))).is_unique
    cycle4_steps.is_unique
}

/// The target list of the canonical four-cycle has length four.
theorem cycle4_steps_length_four {
    cycle4_steps.length = Nat.4
} by {
    List.nil[Nat].length = Nat.0
    List.cons(Nat.0, List.nil[Nat]).length = Nat.1
    List.cons(Nat.3, List.cons(Nat.0, List.nil[Nat])).length = Nat.2
    List.cons(Nat.2, List.cons(Nat.3, List.cons(Nat.0, List.nil[Nat]))).length = Nat.3
    List.cons(Nat.1, List.cons(Nat.2, List.cons(Nat.3, List.cons(Nat.0, List.nil[Nat])))).length = Nat.4
    cycle4_steps.length = Nat.4
}

/// A vertex of the canonical four-cycle is one of the four labels.
theorem cycle4_vertices_cases(x: Nat) {
    simple_graph_walk_vertices(Nat.0, cycle4_steps).contains(x) implies
        (x = Nat.0 or x = Nat.1 or x = Nat.2 or x = Nat.3)
} by {
    if simple_graph_walk_vertices(Nat.0, cycle4_steps).contains(x) {
        simple_graph_walk_vertices(Nat.0, cycle4_steps) = List.cons(Nat.0, cycle4_steps)
        cons_contains_cases(Nat.0, cycle4_steps, x)
        List.cons(Nat.0, cycle4_steps).contains(x) implies (Nat.0 = x or cycle4_steps.contains(x))
        Nat.0 = x or cycle4_steps.contains(x)
        if Nat.0 != x {
            cycle4_steps.contains(x)
            cycle4_steps = List.cons(Nat.1, List.cons(Nat.2, List.cons(Nat.3, List.cons(Nat.0, List.nil[Nat]))))
            cons_contains_cases(Nat.1, List.cons(Nat.2, List.cons(Nat.3, List.cons(Nat.0, List.nil[Nat]))), x)
            List.cons(Nat.1, List.cons(Nat.2, List.cons(Nat.3, List.cons(Nat.0, List.nil[Nat])))).contains(x) implies (Nat.1 = x or List.cons(Nat.2, List.cons(Nat.3, List.cons(Nat.0, List.nil[Nat]))).contains(x))
            Nat.1 = x or List.cons(Nat.2, List.cons(Nat.3, List.cons(Nat.0, List.nil[Nat]))).contains(x)
            if Nat.1 != x {
                List.cons(Nat.2, List.cons(Nat.3, List.cons(Nat.0, List.nil[Nat]))).contains(x)
                cons_contains_cases(Nat.2, List.cons(Nat.3, List.cons(Nat.0, List.nil[Nat])), x)
                List.cons(Nat.2, List.cons(Nat.3, List.cons(Nat.0, List.nil[Nat]))).contains(x) implies (Nat.2 = x or List.cons(Nat.3, List.cons(Nat.0, List.nil[Nat])).contains(x))
                Nat.2 = x or List.cons(Nat.3, List.cons(Nat.0, List.nil[Nat])).contains(x)
                if Nat.2 != x {
                    List.cons(Nat.3, List.cons(Nat.0, List.nil[Nat])).contains(x)
                    cons_contains_cases(Nat.3, List.cons(Nat.0, List.nil[Nat]), x)
                    List.cons(Nat.3, List.cons(Nat.0, List.nil[Nat])).contains(x) implies (Nat.3 = x or List.cons(Nat.0, List.nil[Nat]).contains(x))
                    Nat.3 = x or List.cons(Nat.0, List.nil[Nat]).contains(x)
                    if Nat.3 != x {
                        List.cons(Nat.0, List.nil[Nat]).contains(x)
                        singleton_contains_imp_eq(Nat.0, x)
                        List.singleton(Nat.0).contains(x) implies x = Nat.0
                        List.singleton(Nat.0) = List.cons(Nat.0, List.nil[Nat])
                        x = Nat.0
                    }
                    x = Nat.0 or x = Nat.1 or x = Nat.2 or x = Nat.3
                }
                x = Nat.0 or x = Nat.1 or x = Nat.2 or x = Nat.3
            }
            x = Nat.0 or x = Nat.1 or x = Nat.2 or x = Nat.3
        }
        x = Nat.0 or x = Nat.1 or x = Nat.2 or x = Nat.3
    }
}
/// Every vertex of the canonical four-cycle lies in the four-element vertex set.
theorem cycle4_vertices_in_set(x: Nat) {
    simple_graph_walk_vertices(Nat.0, cycle4_steps).contains(x) implies four_vertices.contains(x)
} by {
    if simple_graph_walk_vertices(Nat.0, cycle4_steps).contains(x) {
        cycle4_vertices_cases(x)
        (x = Nat.0 or x = Nat.1 or x = Nat.2 or x = Nat.3)
        if x = Nat.0 {
            four_labels_lt
            (Nat.0 < Nat.4 and Nat.1 < Nat.4 and Nat.2 < Nat.4 and Nat.3 < Nat.4)
            Nat.0 < Nat.4
            four_vertices_contains(Nat.0)
            four_vertices.contains(Nat.0)
            four_vertices.contains(x)
        }
        if x = Nat.1 {
            four_labels_lt
            (Nat.0 < Nat.4 and Nat.1 < Nat.4 and Nat.2 < Nat.4 and Nat.3 < Nat.4)
            Nat.1 < Nat.4
            four_vertices_contains(Nat.1)
            four_vertices.contains(Nat.1)
            four_vertices.contains(x)
        }
        if x = Nat.2 {
            four_labels_lt
            (Nat.0 < Nat.4 and Nat.1 < Nat.4 and Nat.2 < Nat.4 and Nat.3 < Nat.4)
            Nat.2 < Nat.4
            four_vertices_contains(Nat.2)
            four_vertices.contains(Nat.2)
            four_vertices.contains(x)
        }
        if x = Nat.3 {
            four_labels_lt
            (Nat.0 < Nat.4 and Nat.1 < Nat.4 and Nat.2 < Nat.4 and Nat.3 < Nat.4)
            Nat.3 < Nat.4
            four_vertices_contains(Nat.3)
            four_vertices.contains(Nat.3)
            four_vertices.contains(x)
        }
        four_vertices.contains(x)
    }
}

/// The canonical four-cycle is a cycle of the four-cycle graph: a closed walk with four
/// distinct targets.
theorem cycle4_is_cycle {
    simple_graph_cycle(cycle_graph(Nat.4), Nat.0, cycle4_steps) and
    Nat.4 <= cycle4_steps.length and
    (forall(x: Nat) { simple_graph_walk_vertices(Nat.0, cycle4_steps).contains(x) implies four_vertices.contains(x) })
} by {
    simple_graph_walk_nil(cycle_graph(Nat.4), Nat.0, Nat.0)
    simple_graph_walk(cycle_graph(Nat.4), Nat.0, Nat.0, List.nil[Nat])
    four_labels_ordered
    (Nat.0 < Nat.1 and Nat.1 < Nat.2 and Nat.2 < Nat.3 and Nat.3 < Nat.4)
    Nat.2 < Nat.3
    Nat.2 <= Nat.3
    cycle_graph_adj_wrap(Nat.3)
    cycle_graph(Nat.4).adj(Nat.3, Nat.0)
    simple_graph_walk_cons_iff(cycle_graph(Nat.4), Nat.3, Nat.0, Nat.0, List.nil[Nat])
    simple_graph_walk(cycle_graph(Nat.4), Nat.3, Nat.0, List.cons(Nat.0, List.nil[Nat])) =
        (cycle_graph(Nat.4).adj(Nat.3, Nat.0) and simple_graph_walk(cycle_graph(Nat.4), Nat.0, Nat.0, List.nil[Nat]))
    simple_graph_walk(cycle_graph(Nat.4), Nat.3, Nat.0, List.cons(Nat.0, List.nil[Nat]))
    four_labels_lt
    (Nat.0 < Nat.4 and Nat.1 < Nat.4 and Nat.2 < Nat.4 and Nat.3 < Nat.4)
    Nat.3 < Nat.4
    cycle_graph_adj_suc(Nat.4, Nat.2)
    cycle_graph(Nat.4).adj(Nat.2, Nat.3)
    simple_graph_walk_cons_iff(cycle_graph(Nat.4), Nat.2, Nat.0, Nat.3, List.cons(Nat.0, List.nil[Nat]))
    simple_graph_walk(cycle_graph(Nat.4), Nat.2, Nat.0, List.cons(Nat.3, List.cons(Nat.0, List.nil[Nat]))) =
        (cycle_graph(Nat.4).adj(Nat.2, Nat.3) and simple_graph_walk(cycle_graph(Nat.4), Nat.3, Nat.0, List.cons(Nat.0, List.nil[Nat])))
    simple_graph_walk(cycle_graph(Nat.4), Nat.2, Nat.0, List.cons(Nat.3, List.cons(Nat.0, List.nil[Nat])))
    Nat.2 < Nat.4
    cycle_graph_adj_suc(Nat.4, Nat.1)
    cycle_graph(Nat.4).adj(Nat.1, Nat.2)
    simple_graph_walk_cons_iff(cycle_graph(Nat.4), Nat.1, Nat.0, Nat.2, List.cons(Nat.3, List.cons(Nat.0, List.nil[Nat])))
    simple_graph_walk(cycle_graph(Nat.4), Nat.1, Nat.0, List.cons(Nat.2, List.cons(Nat.3, List.cons(Nat.0, List.nil[Nat])))) =
        (cycle_graph(Nat.4).adj(Nat.1, Nat.2) and simple_graph_walk(cycle_graph(Nat.4), Nat.2, Nat.0, List.cons(Nat.3, List.cons(Nat.0, List.nil[Nat]))))
    simple_graph_walk(cycle_graph(Nat.4), Nat.1, Nat.0, List.cons(Nat.2, List.cons(Nat.3, List.cons(Nat.0, List.nil[Nat]))))
    Nat.1 < Nat.4
    cycle_graph_adj_suc(Nat.4, Nat.0)
    cycle_graph(Nat.4).adj(Nat.0, Nat.1)
    simple_graph_walk_cons_iff(cycle_graph(Nat.4), Nat.0, Nat.0, Nat.1, List.cons(Nat.2, List.cons(Nat.3, List.cons(Nat.0, List.nil[Nat]))))
    simple_graph_walk(cycle_graph(Nat.4), Nat.0, Nat.0, List.cons(Nat.1, List.cons(Nat.2, List.cons(Nat.3, List.cons(Nat.0, List.nil[Nat]))))) =
        (cycle_graph(Nat.4).adj(Nat.0, Nat.1) and simple_graph_walk(cycle_graph(Nat.4), Nat.1, Nat.0, List.cons(Nat.2, List.cons(Nat.3, List.cons(Nat.0, List.nil[Nat])))))
    simple_graph_walk(cycle_graph(Nat.4), Nat.0, Nat.0, cycle4_steps)
    cycle4_steps_length_four
    cycle4_steps.length = Nat.4
    Nat.4 > Nat.0
    cycle4_steps.length > Nat.0
    simple_graph_closed_walk(cycle_graph(Nat.4), Nat.0, cycle4_steps)
    cycle4_steps_unique
    cycle4_steps.is_unique
    Nat.4 <= Nat.4
    Nat.4 <= cycle4_steps.length
    simple_graph_cycle(cycle_graph(Nat.4), Nat.0, cycle4_steps) =
        (simple_graph_closed_walk(cycle_graph(Nat.4), Nat.0, cycle4_steps) and
            Nat.3 <= cycle4_steps.length and cycle4_steps.is_unique)
    Nat.3 <= cycle4_steps.length
    simple_graph_cycle(cycle_graph(Nat.4), Nat.0, cycle4_steps)
    forall(x: Nat) {
        if simple_graph_walk_vertices(Nat.0, cycle4_steps).contains(x) {
            cycle4_vertices_in_set(x)
            four_vertices.contains(x)
        }
        (simple_graph_walk_vertices(Nat.0, cycle4_steps).contains(x) implies four_vertices.contains(x))
    }
    simple_graph_cycle(cycle_graph(Nat.4), Nat.0, cycle4_steps) and
    Nat.4 <= cycle4_steps.length and
    (forall(x: Nat) { simple_graph_walk_vertices(Nat.0, cycle4_steps).contains(x) implies four_vertices.contains(x) })
}

/// The canonical four-cycle has no chord: every adjacency of the four-cycle graph between two
/// of its vertices is traversed by the cycle itself.
theorem cycle4_has_no_chord {
    not has_chord(cycle_graph(Nat.4), Nat.0, cycle4_steps)
} by {
    if has_chord(cycle_graph(Nat.4), Nat.0, cycle4_steps) {
        has_chord(cycle_graph(Nat.4), Nat.0, cycle4_steps) = exists(x: Nat, y: Nat) {
            simple_graph_walk_vertices(Nat.0, cycle4_steps).contains(x) and
            simple_graph_walk_vertices(Nat.0, cycle4_steps).contains(y) and
            x != y and cycle_graph(Nat.4).adj(x, y) and
            not is_walk_edge(Nat.0, cycle4_steps, x, y) and
            not is_walk_edge(Nat.0, cycle4_steps, y, x)
        }
        let (x: Nat, y: Nat) satisfy {
            simple_graph_walk_vertices(Nat.0, cycle4_steps).contains(x) and
            simple_graph_walk_vertices(Nat.0, cycle4_steps).contains(y) and
            x != y and cycle_graph(Nat.4).adj(x, y) and
            not is_walk_edge(Nat.0, cycle4_steps, x, y) and
            not is_walk_edge(Nat.0, cycle4_steps, y, x)
        }
        cycle4_vertices_cases(x)
        x = Nat.0 or x = Nat.1 or x = Nat.2 or x = Nat.3
        cycle4_vertices_cases(y)
        y = Nat.0 or y = Nat.1 or y = Nat.2 or y = Nat.3
        if x = Nat.0 {
            cycle_graph(Nat.4).adj(Nat.0, y)
            cycle4_adj_zero_imp_walk_edge(y)
            (y = Nat.0 or y = Nat.1 or y = Nat.2 or y = Nat.3) and cycle_graph(Nat.4).adj(Nat.0, y) implies (is_walk_edge(Nat.0, cycle4_steps, Nat.0, y) or is_walk_edge(Nat.0, cycle4_steps, y, Nat.0))
            (y = Nat.0 or y = Nat.1 or y = Nat.2 or y = Nat.3) and cycle_graph(Nat.4).adj(Nat.0, y)
            is_walk_edge(Nat.0, cycle4_steps, Nat.0, y) or is_walk_edge(Nat.0, cycle4_steps, y, Nat.0)
            if is_walk_edge(Nat.0, cycle4_steps, Nat.0, y) or is_walk_edge(Nat.0, cycle4_steps, y, Nat.0) {
                if is_walk_edge(Nat.0, cycle4_steps, Nat.0, y) {
                    not is_walk_edge(Nat.0, cycle4_steps, x, y)
                    false
                }
                if is_walk_edge(Nat.0, cycle4_steps, y, Nat.0) {
                    not is_walk_edge(Nat.0, cycle4_steps, y, x)
                    false
                }
                false
            }
            false
        }
        if x = Nat.1 {
            cycle_graph(Nat.4).adj(Nat.1, y)
            cycle4_adj_one_imp_walk_edge(y)
            (y = Nat.0 or y = Nat.1 or y = Nat.2 or y = Nat.3) and cycle_graph(Nat.4).adj(Nat.1, y) implies (is_walk_edge(Nat.0, cycle4_steps, Nat.1, y) or is_walk_edge(Nat.0, cycle4_steps, y, Nat.1))
            (y = Nat.0 or y = Nat.1 or y = Nat.2 or y = Nat.3) and cycle_graph(Nat.4).adj(Nat.1, y)
            is_walk_edge(Nat.0, cycle4_steps, Nat.1, y) or is_walk_edge(Nat.0, cycle4_steps, y, Nat.1)
            if is_walk_edge(Nat.0, cycle4_steps, Nat.1, y) or is_walk_edge(Nat.0, cycle4_steps, y, Nat.1) {
                if is_walk_edge(Nat.0, cycle4_steps, Nat.1, y) {
                    not is_walk_edge(Nat.0, cycle4_steps, x, y)
                    false
                }
                if is_walk_edge(Nat.0, cycle4_steps, y, Nat.1) {
                    not is_walk_edge(Nat.0, cycle4_steps, y, x)
                    false
                }
                false
            }
            false
        }
        if x = Nat.2 {
            cycle_graph(Nat.4).adj(Nat.2, y)
            cycle4_adj_two_imp_walk_edge(y)
            (y = Nat.0 or y = Nat.1 or y = Nat.2 or y = Nat.3) and cycle_graph(Nat.4).adj(Nat.2, y) implies (is_walk_edge(Nat.0, cycle4_steps, Nat.2, y) or is_walk_edge(Nat.0, cycle4_steps, y, Nat.2))
            (y = Nat.0 or y = Nat.1 or y = Nat.2 or y = Nat.3) and cycle_graph(Nat.4).adj(Nat.2, y)
            is_walk_edge(Nat.0, cycle4_steps, Nat.2, y) or is_walk_edge(Nat.0, cycle4_steps, y, Nat.2)
            if is_walk_edge(Nat.0, cycle4_steps, Nat.2, y) or is_walk_edge(Nat.0, cycle4_steps, y, Nat.2) {
                if is_walk_edge(Nat.0, cycle4_steps, Nat.2, y) {
                    not is_walk_edge(Nat.0, cycle4_steps, x, y)
                    false
                }
                if is_walk_edge(Nat.0, cycle4_steps, y, Nat.2) {
                    not is_walk_edge(Nat.0, cycle4_steps, y, x)
                    false
                }
                false
            }
            false
        }
        if x = Nat.3 {
            cycle_graph(Nat.4).adj(Nat.3, y)
            cycle4_adj_three_imp_walk_edge(y)
            (y = Nat.0 or y = Nat.1 or y = Nat.2 or y = Nat.3) and cycle_graph(Nat.4).adj(Nat.3, y) implies (is_walk_edge(Nat.0, cycle4_steps, Nat.3, y) or is_walk_edge(Nat.0, cycle4_steps, y, Nat.3))
            (y = Nat.0 or y = Nat.1 or y = Nat.2 or y = Nat.3) and cycle_graph(Nat.4).adj(Nat.3, y)
            is_walk_edge(Nat.0, cycle4_steps, Nat.3, y) or is_walk_edge(Nat.0, cycle4_steps, y, Nat.3)
            if is_walk_edge(Nat.0, cycle4_steps, Nat.3, y) or is_walk_edge(Nat.0, cycle4_steps, y, Nat.3) {
                if is_walk_edge(Nat.0, cycle4_steps, Nat.3, y) {
                    not is_walk_edge(Nat.0, cycle4_steps, x, y)
                    false
                }
                if is_walk_edge(Nat.0, cycle4_steps, y, Nat.3) {
                    not is_walk_edge(Nat.0, cycle4_steps, y, x)
                    false
                }
                false
            }
            false
        }
        false
    }
    not has_chord(cycle_graph(Nat.4), Nat.0, cycle4_steps)
}

/// The four-cycle graph on four vertices is not chordal: its canonical four-cycle has no
/// chord.
theorem cycle4_not_chordal {
    not is_chordal(cycle_graph(Nat.4), four_vertices)
} by {
    if is_chordal(cycle_graph(Nat.4), four_vertices) {
        cycle4_is_cycle
        simple_graph_cycle(cycle_graph(Nat.4), Nat.0, cycle4_steps) and
        Nat.4 <= cycle4_steps.length and
        (forall(x: Nat) { simple_graph_walk_vertices(Nat.0, cycle4_steps).contains(x) implies four_vertices.contains(x) })
        simple_graph_cycle(cycle_graph(Nat.4), Nat.0, cycle4_steps)
        Nat.4 <= cycle4_steps.length
        forall(x: Nat) { simple_graph_walk_vertices(Nat.0, cycle4_steps).contains(x) implies four_vertices.contains(x) }
        is_chordal_apply(cycle_graph(Nat.4), four_vertices, Nat.0, cycle4_steps)
        is_chordal(cycle_graph(Nat.4), four_vertices) and simple_graph_cycle(cycle_graph(Nat.4), Nat.0, cycle4_steps) and Nat.4 <= cycle4_steps.length and (forall(x: Nat) { simple_graph_walk_vertices(Nat.0, cycle4_steps).contains(x) implies four_vertices.contains(x) }) implies has_chord(cycle_graph(Nat.4), Nat.0, cycle4_steps)
        has_chord(cycle_graph(Nat.4), Nat.0, cycle4_steps)
        cycle4_has_no_chord
        not has_chord(cycle_graph(Nat.4), Nat.0, cycle4_steps)
        false
    }
    not is_chordal(cycle_graph(Nat.4), four_vertices)
}

/// In a closed walk whose target list begins with three further targets, the remainder of the
/// list contains the start: the walk returns to its start at the end.
theorem closed_walk_rest_contains_start[V](g: SimpleGraph[V], start: V, steps: List[V], c1: V, c2: V, c3: V, rest: List[V]) {
    simple_graph_walk(g, start, start, steps) and
    steps = List.cons(c1, List.cons(c2, List.cons(c3, rest))) and
    Nat.4 <= steps.length implies rest.contains(start)
} by {
    if simple_graph_walk(g, start, start, steps) and
        steps = List.cons(c1, List.cons(c2, List.cons(c3, rest))) and
        Nat.4 <= steps.length {
        simple_graph_walk(g, start, start, steps)
        steps = List.cons(c1, List.cons(c2, List.cons(c3, rest)))
        Nat.4 <= steps.length
        simple_graph_walk_cons_iff(g, start, start, c1, List.cons(c2, List.cons(c3, rest)))
        simple_graph_walk(g, start, start, List.cons(c1, List.cons(c2, List.cons(c3, rest)))) =
            (g.adj(start, c1) and simple_graph_walk(g, c1, start, List.cons(c2, List.cons(c3, rest))))
        simple_graph_walk(g, c1, start, List.cons(c2, List.cons(c3, rest)))
        simple_graph_walk_cons_iff(g, c1, start, c2, List.cons(c3, rest))
        simple_graph_walk(g, c1, start, List.cons(c2, List.cons(c3, rest))) =
            (g.adj(c1, c2) and simple_graph_walk(g, c2, start, List.cons(c3, rest)))
        simple_graph_walk(g, c2, start, List.cons(c3, rest))
        simple_graph_walk_cons_iff(g, c2, start, c3, rest)
        simple_graph_walk(g, c2, start, List.cons(c3, rest)) =
            (g.adj(c2, c3) and simple_graph_walk(g, c3, start, rest))
        simple_graph_walk(g, c3, start, rest)
        List.cons(c3, rest).length = rest.length + Nat.1
        List.cons(c2, List.cons(c3, rest)).length = List.cons(c3, rest).length + Nat.1
        List.cons(c2, List.cons(c3, rest)).length = rest.length + Nat.1 + Nat.1
        List.cons(c1, List.cons(c2, List.cons(c3, rest))).length = List.cons(c2, List.cons(c3, rest)).length + Nat.1
        List.cons(c1, List.cons(c2, List.cons(c3, rest))).length = rest.length + Nat.3
        steps.length = List.cons(c1, List.cons(c2, List.cons(c3, rest))).length
        Nat.4 <= rest.length + Nat.3
        if rest.length = Nat.0 {
            rest.length + Nat.3 = Nat.3
            Nat.4 <= Nat.3
            lt_suc(Nat.3)
            Nat.3 < Nat.4
            lte_and_lt(Nat.4, Nat.3, Nat.4)
            Nat.4 < Nat.4
            false
        }
        if rest.length != Nat.0 {
            Nat.0 < rest.length
        }
        Nat.0 < rest.length
        simple_graph_walk_contains_finish(g, c3, start, rest)
        simple_graph_walk(g, c3, start, rest) implies (rest = List.nil[V] or rest.contains(start))
        rest = List.nil[V] or rest.contains(start)
        if rest = List.nil[V] {
            rest.length = Nat.0
            false
        }
        if rest != List.nil[V] {
            if rest = List.nil[V] or rest.contains(start) {
                if rest = List.nil[V] {
                    false
                }
                if rest != List.nil[V] {
                    rest.contains(start)
                }
                rest.contains(start)
            }
            rest.contains(start)
        }
        rest.contains(start)
    }
}

/// A disjunction whose left disjunct is false reduces to its right disjunct.
theorem or_false_left_imp_right(a: Bool, b: Bool) {
    not a and (a or b) implies b
} by {
    if not a and (a or b) {
        if a {
            a
            not a
            false
        }
        if not a {
            b
        }
        b
    }
}

/// The first and third targets of a list of four or more distinct targets are not joined by a
/// walk edge: the only edges of the walk leave each target for its immediate successor.
theorem cycle_third_vertex_no_chord_edges[V](start: V, steps: List[V], c1: V, c2: V, c3: V, rest: List[V]) {
    steps = List.cons(c1, List.cons(c2, List.cons(c3, rest))) and
    steps.is_unique and
    start != c3 and
    (forall(w: V) { rest.contains(w) implies w != c3 }) and
    (forall(w: V) { rest.contains(w) implies w != c1 }) implies (not is_walk_edge(start, steps, c1, c3) and not is_walk_edge(start, steps, c3, c1))
} by {
    if steps = List.cons(c1, List.cons(c2, List.cons(c3, rest))) and steps.is_unique and start != c3 and
        (forall(w: V) { rest.contains(w) implies w != c3 }) and
        (forall(w: V) { rest.contains(w) implies w != c1 }) {
        steps = List.cons(c1, List.cons(c2, List.cons(c3, rest)))
        steps.is_unique
        start != c3
        (forall(w: V) { rest.contains(w) implies w != c3 })
        (forall(w: V) { rest.contains(w) implies w != c1 })
        cons_contains_head(c2, List.cons(c3, rest))
        List.cons(c2, List.cons(c3, rest)).contains(c2)
        unique_cons_head_fresh(c1, List.cons(c2, List.cons(c3, rest)), c2)
        c1 != c2
        cons_contains_of_tail_contains(c2, List.cons(c3, rest), c3)
        List.cons(c2, List.cons(c3, rest)).contains(c3)
        unique_cons_head_fresh(c1, List.cons(c2, List.cons(c3, rest)), c3)
        c1 != c3
        unique_cons_parts(c1, List.cons(c2, List.cons(c3, rest)))
        List.cons(c1, List.cons(c2, List.cons(c3, rest))).is_unique implies (List.cons(c2, List.cons(c3, rest)).is_unique and not List.cons(c2, List.cons(c3, rest)).contains(c1))
        List.cons(c2, List.cons(c3, rest)).is_unique
        cons_contains_head(c3, rest)
        List.cons(c3, rest).contains(c3)
        unique_cons_head_fresh(c2, List.cons(c3, rest), c3)
        c2 != c3
        if is_walk_edge(start, steps, c1, c3) {
            is_walk_edge_cons(start, c1, List.cons(c2, List.cons(c3, rest)), c1, c3)
            is_walk_edge(start, List.cons(c1, List.cons(c2, List.cons(c3, rest))), c1, c3) =
                ((start = c1 and c1 = c3) or is_walk_edge(c1, List.cons(c2, List.cons(c3, rest)), c1, c3))
            (start = c1 and c1 = c3) or is_walk_edge(c1, List.cons(c2, List.cons(c3, rest)), c1, c3)
            if start = c1 and c1 = c3 {
                c1 = c3
                c1 != c3
                false
            }
            not (start = c1 and c1 = c3)
            or_false_left_imp_right((start = c1 and c1 = c3), is_walk_edge(c1, List.cons(c2, List.cons(c3, rest)), c1, c3))
            (not (start = c1 and c1 = c3) and ((start = c1 and c1 = c3) or is_walk_edge(c1, List.cons(c2, List.cons(c3, rest)), c1, c3))) implies is_walk_edge(c1, List.cons(c2, List.cons(c3, rest)), c1, c3)
            is_walk_edge(c1, List.cons(c2, List.cons(c3, rest)), c1, c3)
            is_walk_edge_cons(c1, c2, List.cons(c3, rest), c1, c3)
            is_walk_edge(c1, List.cons(c2, List.cons(c3, rest)), c1, c3) =
                ((c1 = c1 and c2 = c3) or is_walk_edge(c2, List.cons(c3, rest), c1, c3))
            (c1 = c1 and c2 = c3) or is_walk_edge(c2, List.cons(c3, rest), c1, c3)
            if c1 = c1 and c2 = c3 {
                c2 = c3
                c2 != c3
                false
            }
            not (c1 = c1 and c2 = c3)
            or_false_left_imp_right((c1 = c1 and c2 = c3), is_walk_edge(c2, List.cons(c3, rest), c1, c3))
            (not (c1 = c1 and c2 = c3) and ((c1 = c1 and c2 = c3) or is_walk_edge(c2, List.cons(c3, rest), c1, c3))) implies is_walk_edge(c2, List.cons(c3, rest), c1, c3)
            is_walk_edge(c2, List.cons(c3, rest), c1, c3)
            is_walk_edge_cons(c2, c3, rest, c1, c3)
            is_walk_edge(c2, List.cons(c3, rest), c1, c3) =
                ((c2 = c1 and c3 = c3) or is_walk_edge(c3, rest, c1, c3))
            (c2 = c1 and c3 = c3) or is_walk_edge(c3, rest, c1, c3)
            if c2 = c1 and c3 = c3 {
                c2 = c1
                c2 != c1
                false
            }
            not (c2 = c1 and c3 = c3)
            or_false_left_imp_right((c2 = c1 and c3 = c3), is_walk_edge(c3, rest, c1, c3))
            (not (c2 = c1 and c3 = c3) and ((c2 = c1 and c3 = c3) or is_walk_edge(c3, rest, c1, c3))) implies is_walk_edge(c3, rest, c1, c3)
            is_walk_edge(c3, rest, c1, c3)
            is_walk_edge_false_of_second_absent(c3, rest, c1, c3)
            (forall(w: V) { rest.contains(w) implies w != c3 }) implies not is_walk_edge(c3, rest, c1, c3)
            not is_walk_edge(c3, rest, c1, c3)
            false
        }
        not is_walk_edge(start, steps, c1, c3)
        if is_walk_edge(start, steps, c3, c1) {
            is_walk_edge_cons(start, c1, List.cons(c2, List.cons(c3, rest)), c3, c1)
            is_walk_edge(start, List.cons(c1, List.cons(c2, List.cons(c3, rest))), c3, c1) =
                ((start = c3 and c1 = c1) or is_walk_edge(c1, List.cons(c2, List.cons(c3, rest)), c3, c1))
            (start = c3 and c1 = c1) or is_walk_edge(c1, List.cons(c2, List.cons(c3, rest)), c3, c1)
            if start = c3 and c1 = c1 {
                start = c3
                start != c3
                false
            }
            not (start = c3 and c1 = c1)
            or_false_left_imp_right((start = c3 and c1 = c1), is_walk_edge(c1, List.cons(c2, List.cons(c3, rest)), c3, c1))
            (not (start = c3 and c1 = c1) and ((start = c3 and c1 = c1) or is_walk_edge(c1, List.cons(c2, List.cons(c3, rest)), c3, c1))) implies is_walk_edge(c1, List.cons(c2, List.cons(c3, rest)), c3, c1)
            is_walk_edge(c1, List.cons(c2, List.cons(c3, rest)), c3, c1)
            is_walk_edge_cons(c1, c2, List.cons(c3, rest), c3, c1)
            is_walk_edge(c1, List.cons(c2, List.cons(c3, rest)), c3, c1) =
                ((c1 = c3 and c2 = c1) or is_walk_edge(c2, List.cons(c3, rest), c3, c1))
            (c1 = c3 and c2 = c1) or is_walk_edge(c2, List.cons(c3, rest), c3, c1)
            if c1 = c3 and c2 = c1 {
                c1 = c3
                c1 != c3
                false
            }
            not (c1 = c3 and c2 = c1)
            or_false_left_imp_right((c1 = c3 and c2 = c1), is_walk_edge(c2, List.cons(c3, rest), c3, c1))
            (not (c1 = c3 and c2 = c1) and ((c1 = c3 and c2 = c1) or is_walk_edge(c2, List.cons(c3, rest), c3, c1))) implies is_walk_edge(c2, List.cons(c3, rest), c3, c1)
            is_walk_edge(c2, List.cons(c3, rest), c3, c1)
            is_walk_edge_cons(c2, c3, rest, c3, c1)
            is_walk_edge(c2, List.cons(c3, rest), c3, c1) =
                ((c2 = c3 and c3 = c1) or is_walk_edge(c3, rest, c3, c1))
            (c2 = c3 and c3 = c1) or is_walk_edge(c3, rest, c3, c1)
            if c2 = c3 and c3 = c1 {
                c2 = c3
                c2 != c3
                false
            }
            not (c2 = c3 and c3 = c1)
            or_false_left_imp_right((c2 = c3 and c3 = c1), is_walk_edge(c3, rest, c3, c1))
            (not (c2 = c3 and c3 = c1) and ((c2 = c3 and c3 = c1) or is_walk_edge(c3, rest, c3, c1))) implies is_walk_edge(c3, rest, c3, c1)
            is_walk_edge(c3, rest, c3, c1)
            is_walk_edge_false_of_second_absent(c3, rest, c3, c1)
            (forall(w: V) { rest.contains(w) implies w != c1 }) implies not is_walk_edge(c3, rest, c3, c1)
            not is_walk_edge(c3, rest, c3, c1)
            false
        }
        not is_walk_edge(start, steps, c3, c1)
        not is_walk_edge(start, steps, c1, c3) and not is_walk_edge(start, steps, c3, c1)
    }
}

/// Every cycle of length at least four in a complete graph has a chord: the first and third
/// vertices of the cycle are distinct, adjacent, and not consecutive along it.
theorem complete_graph_is_chordal[V](s: FiniteSet[V]) {
    is_chordal(complete_graph[V], s)
} by {
    is_chordal_intro(complete_graph[V], s)
    (forall(start: V, steps: List[V]) {
        simple_graph_cycle(complete_graph[V], start, steps) and Nat.4 <= steps.length and
        (forall(x: V) { simple_graph_walk_vertices(start, steps).contains(x) implies s.contains(x) }) implies has_chord(complete_graph[V], start, steps)
    }) implies is_chordal(complete_graph[V], s)
    forall(start: V, steps: List[V]) {
        if simple_graph_cycle(complete_graph[V], start, steps) and Nat.4 <= steps.length and
            (forall(x: V) { simple_graph_walk_vertices(start, steps).contains(x) implies s.contains(x) }) {
            simple_graph_cycle(complete_graph[V], start, steps)
            Nat.4 <= steps.length
            (forall(x: V) { simple_graph_walk_vertices(start, steps).contains(x) implies s.contains(x) })
            list_four_of_length(steps)
            Nat.4 <= steps.length implies exists(c1: V, c2: V, c3: V, rest: List[V]) {
                steps = List.cons(c1, List.cons(c2, List.cons(c3, rest)))
            }
            exists(c1: V, c2: V, c3: V, rest: List[V]) {
                steps = List.cons(c1, List.cons(c2, List.cons(c3, rest)))
            }
            let (c1: V, c2: V, c3: V, rest: List[V]) satisfy {
                steps = List.cons(c1, List.cons(c2, List.cons(c3, rest)))
            }
            simple_graph_cycle(complete_graph[V], start, steps) =
                (simple_graph_closed_walk(complete_graph[V], start, steps) and
                    Nat.3 <= steps.length and steps.is_unique)
            simple_graph_closed_walk(complete_graph[V], start, steps)
            steps.is_unique
            simple_graph_closed_walk_is_walk(complete_graph[V], start, steps)
            simple_graph_walk(complete_graph[V], start, start, steps)
            closed_walk_rest_contains_start(complete_graph[V], start, steps, c1, c2, c3, rest)
            simple_graph_walk(complete_graph[V], start, start, steps) and steps = List.cons(c1, List.cons(c2, List.cons(c3, rest))) and Nat.4 <= steps.length implies rest.contains(start)
            rest.contains(start)
            unique_cons_parts(c1, List.cons(c2, List.cons(c3, rest)))
            List.cons(c1, List.cons(c2, List.cons(c3, rest))).is_unique implies (List.cons(c2, List.cons(c3, rest)).is_unique and not List.cons(c2, List.cons(c3, rest)).contains(c1))
            List.cons(c2, List.cons(c3, rest)).is_unique
            unique_cons_parts(c2, List.cons(c3, rest))
            List.cons(c2, List.cons(c3, rest)).is_unique implies (List.cons(c3, rest).is_unique and not List.cons(c3, rest).contains(c2))
            List.cons(c3, rest).is_unique
            unique_cons_head_fresh(c3, rest, start)
            List.cons(c3, rest).is_unique and rest.contains(start) implies c3 != start
            c3 != start
            start != c3
            unique_cons_head_not_in_tail(c3, rest)
            List.cons(c3, rest).is_unique implies not rest.contains(c3)
            not rest.contains(c3)
            forall(w: V) {
                if rest.contains(w) {
                    if w = c3 {
                        rest.contains(c3)
                        not rest.contains(c3)
                        false
                    }
                    w != c3
                }
                rest.contains(w) implies w != c3
            }
            not List.cons(c2, List.cons(c3, rest)).contains(c1)
            forall(w: V) {
                if rest.contains(w) {
                    if w = c1 {
                        cons_contains_of_tail_contains(c2, List.cons(c3, rest), c1)
                        List.cons(c2, List.cons(c3, rest)).contains(c1)
                        not List.cons(c2, List.cons(c3, rest)).contains(c1)
                        false
                    }
                    w != c1
                }
                rest.contains(w) implies w != c1
            }
            cycle_third_vertex_no_chord_edges(start, steps, c1, c2, c3, rest)
            (steps = List.cons(c1, List.cons(c2, List.cons(c3, rest))) and steps.is_unique and start != c3 and
                (forall(w: V) { rest.contains(w) implies w != c3 }) and
                (forall(w: V) { rest.contains(w) implies w != c1 })) implies (not is_walk_edge(start, steps, c1, c3) and not is_walk_edge(start, steps, c3, c1))
            steps = List.cons(c1, List.cons(c2, List.cons(c3, rest)))
            steps.is_unique
            start != c3
            forall(w: V) { rest.contains(w) implies w != c3 }
            forall(w: V) { rest.contains(w) implies w != c1 }
            steps = List.cons(c1, List.cons(c2, List.cons(c3, rest))) and steps.is_unique
            steps = List.cons(c1, List.cons(c2, List.cons(c3, rest))) and steps.is_unique and start != c3
            steps = List.cons(c1, List.cons(c2, List.cons(c3, rest))) and steps.is_unique and start != c3 and (forall(w: V) { rest.contains(w) implies w != c3 })
            steps = List.cons(c1, List.cons(c2, List.cons(c3, rest))) and steps.is_unique and start != c3 and (forall(w: V) { rest.contains(w) implies w != c3 }) and (forall(w: V) { rest.contains(w) implies w != c1 })
            not is_walk_edge(start, steps, c1, c3) and not is_walk_edge(start, steps, c3, c1)
            not is_walk_edge(start, steps, c1, c3)
            not is_walk_edge(start, steps, c3, c1)
            simple_graph_walk_vertices(start, steps) = List.cons(start, steps)
            cons_contains_of_tail_contains(start, List.cons(c1, List.cons(c2, List.cons(c3, rest))), c1)
            List.cons(start, List.cons(c1, List.cons(c2, List.cons(c3, rest)))).contains(c1)
            simple_graph_walk_vertices(start, steps).contains(c1)
            cons_contains_of_tail_contains(c1, List.cons(c2, List.cons(c3, rest)), c3)
            cons_contains_of_tail_contains(c2, List.cons(c3, rest), c3)
            List.cons(c2, List.cons(c3, rest)).contains(c3)
            List.cons(c1, List.cons(c2, List.cons(c3, rest))).contains(c3)
            List.cons(start, List.cons(c1, List.cons(c2, List.cons(c3, rest)))).contains(c3)
            simple_graph_walk_vertices(start, steps).contains(c3)
            c1 != c3
            complete_graph_adj_iff_ne[V](c1, c3)
            complete_graph[V].adj(c1, c3) = (c1 != c3)
            complete_graph[V].adj(c1, c3)
            has_chord_intro(complete_graph[V], start, steps, c1, c3)
            simple_graph_walk_vertices(start, steps).contains(c1) and
            simple_graph_walk_vertices(start, steps).contains(c3) and
            c1 != c3 and complete_graph[V].adj(c1, c3) and
            not is_walk_edge(start, steps, c1, c3) and
            not is_walk_edge(start, steps, c3, c1) implies has_chord(complete_graph[V], start, steps)
            has_chord(complete_graph[V], start, steps)
        }
        (simple_graph_cycle(complete_graph[V], start, steps) and Nat.4 <= steps.length and
            (forall(x: V) { simple_graph_walk_vertices(start, steps).contains(x) implies s.contains(x) }) implies has_chord(complete_graph[V], start, steps))
    }
    is_chordal(complete_graph[V], s)
}

/// An acyclic graph on `s` has no cycle whose vertices all lie in `s`.
theorem is_acyclic_apply[V](g: SimpleGraph[V], s: FiniteSet[V], start: V, steps: List[V]) {
    not (is_acyclic(g, s) and s.contains(start) and simple_graph_cycle(g, start, steps) and
        (forall(x: V) { steps.contains(x) implies s.contains(x) }))
} by {
    if is_acyclic(g, s) and s.contains(start) and simple_graph_cycle(g, start, steps) and
        (forall(x: V) { steps.contains(x) implies s.contains(x) }) {
        is_acyclic(g, s) = forall(a: V, b: List[V]) {
            s.contains(a) and simple_graph_cycle(g, a, b) and
                (forall(x: V) { b.contains(x) implies s.contains(x) })
                implies false
        }
        s.contains(start) and simple_graph_cycle(g, start, steps) and
            (forall(x: V) { steps.contains(x) implies s.contains(x) })
        false
    }
    not (is_acyclic(g, s) and s.contains(start) and simple_graph_cycle(g, start, steps) and
        (forall(x: V) { steps.contains(x) implies s.contains(x) }))
}

/// A tree is chordal: it has no cycle at all, so the chord condition is vacuous.
theorem tree_is_chordal[V](g: SimpleGraph[V], s: FiniteSet[V]) {
    is_tree(g, s) implies is_chordal(g, s)
} by {
    if is_tree(g, s) {
        is_chordal_intro(g, s)
        (forall(start: V, steps: List[V]) {
            simple_graph_cycle(g, start, steps) and Nat.4 <= steps.length and
            (forall(x: V) { simple_graph_walk_vertices(start, steps).contains(x) implies s.contains(x) })
            implies has_chord(g, start, steps)
        }) implies is_chordal(g, s)
        forall(start: V, steps: List[V]) {
            if simple_graph_cycle(g, start, steps) and Nat.4 <= steps.length and
                (forall(x: V) { simple_graph_walk_vertices(start, steps).contains(x) implies s.contains(x) }) {
                simple_graph_cycle(g, start, steps)
                (forall(x: V) { simple_graph_walk_vertices(start, steps).contains(x) implies s.contains(x) })
                simple_graph_walk_vertices(start, steps) = List.cons(start, steps)
                cons_contains_head(start, steps)
                List.cons(start, steps).contains(start)
                simple_graph_walk_vertices(start, steps).contains(start)
                simple_graph_walk_vertices(start, steps).contains(start) implies s.contains(start)
                s.contains(start)
                forall(x: V) {
                    if steps.contains(x) {
                        cons_contains_of_tail_contains(start, steps, x)
                        List.cons(start, steps).contains(x)
                        simple_graph_walk_vertices(start, steps).contains(x)
                        simple_graph_walk_vertices(start, steps).contains(x) implies s.contains(x)
                        s.contains(x)
                    }
                    steps.contains(x) implies s.contains(x)
                }
                tree_acyclic(g, s)
                is_acyclic(g, s)
                is_acyclic_apply(g, s, start, steps)
                not (is_acyclic(g, s) and s.contains(start) and simple_graph_cycle(g, start, steps) and
                    (forall(x: V) { steps.contains(x) implies s.contains(x) }))
                is_acyclic(g, s) and s.contains(start) and simple_graph_cycle(g, start, steps) and
                    (forall(x: V) { steps.contains(x) implies s.contains(x) })
                false
            }
            (simple_graph_cycle(g, start, steps) and Nat.4 <= steps.length and
                (forall(x: V) { simple_graph_walk_vertices(start, steps).contains(x) implies s.contains(x) })) = false
            false_implies(has_chord(g, start, steps))
            false implies has_chord(g, start, steps)
            (false implies has_chord(g, start, steps)) = true
            (simple_graph_cycle(g, start, steps) and Nat.4 <= steps.length and
                (forall(x: V) { simple_graph_walk_vertices(start, steps).contains(x) implies s.contains(x) })) implies has_chord(g, start, steps)
        }
        is_chordal(g, s)
    }
}

/// The complete graph on the four vertices `{0, 1, 2, 3}` is chordal: a special case of the
/// general theorem above, stated for the concrete vertex set used elsewhere in this file.
theorem complete_graph_four_is_chordal {
    is_chordal(complete_graph[Nat], four_vertices)
} by {
    complete_graph_is_chordal[Nat](four_vertices)
    is_chordal(complete_graph[Nat], four_vertices)
}

// Fulkerson–Gross: a graph is chordal if and only if its vertices admit a perfect elimination
// ordering. The forward direction: a chordal graph always has a simplicial vertex — one whose
// neighbours form a clique — since otherwise walking greedily out of a longest path would
// produce a chordless cycle of length at least four; removing a simplicial vertex keeps the
// graph chordal, and the removal order read backwards is a perfect elimination ordering. The
// reverse direction: given a perfect elimination ordering, take any cycle of length at least
// four and let `v` be its first vertex in the ordering; the two neighbours of `v` on the cycle
// are later neighbours of `v`, hence adjacent, a chord. The statement is recorded here for
// future work; the induction machinery for removing vertices and for reading off "later
// neighbours" from the ordering is not yet in the library.
//
// theorem fulkerson_gross[V](g: SimpleGraph[V], s: FiniteSet[V]) {
//     is_chordal(g, s) = exists(o: List[V]) { is_peo(g, s, o) }
// }
