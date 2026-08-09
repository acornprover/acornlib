from nat import Nat, only_zero_lte_zero, lte_and_lt, lte_trans, lt_suc, lt_trans,
    lt_imp_lte_suc, pos_of_ne_zero, add_one_right, lte_cancel_suc, lte_suc_suc, lte_imp_not_lt,
    lte_add_left, lt_and_lte
from list import List, singleton_unique, singleton_contains_imp_eq, add_contains_right,
    add_contains_or, add_contains_left, add_length
from finite_set import FiniteSet, fs_remove, fs_insert, finite_set_subset_contains,
    finite_set_empty_contains_eq, fs_from_list, finite_set_from_unique_list_cardinality_is_length
from data.finite.finite_set_membership import fs_insert_contains_eq, fs_from_list_contains_eq,
    fs_remove_contains_eq
from data.finite.finite_set_filter import finite_set_filter, finite_set_filter_contains_eq
from data.finite.finite_set_card import fs_card, fs_card_empty, fs_card_eq_of_cardinality_is
from data.finite.finite_set_card_ops import fs_card_insert_of_not_contains, fs_card_remove_of_contains
from data.finite.finite_set_card_compare import fs_card_mono
from data.finite.finite_set_card_members import fs_two_distinct_members, fs_member_of_card_pos
from data.finite.finite_set_subset_intro import fs_subset_eq_intro
from data.nat.nat_bounded_max import is_max, is_max_apply, is_max_is_upper_bound, is_max_intro,
    has_max, is_upper_bound_of, is_upper_bound_of_intro
from data.basic.logic import false_implies, eq_true_elim, not_intro, not_exists_forward,
    not_forall_imp_exists_not, not_implies
from data.list.list_unique_cons import unique_cons_head_not_in_tail, unique_cons_head_fresh,
    unique_cons_parts
from graph.simple_graph import SimpleGraph, induced_subgraph, induced_subgraph_adj_eq,
    induced_subgraph_adj_imp_base_adj, induced_subgraph_adj_imp_left_mem,
    induced_subgraph_adj_imp_right_mem, simple_graph_adj_ne, simple_graph_adj_symmetric
from graph.simple_graph_connectivity import simple_graph_reachable
from graph.simple_graph_walks import simple_graph_walk, simple_graph_closed_walk,
    simple_graph_reachable_has_walk, simple_graph_walk_cons_iff, simple_graph_walk_nil,
    simple_graph_walk_refl, simple_graph_walk_of_adj, simple_graph_walk_vertices,
    simple_graph_walk_append_adj, simple_graph_path, simple_graph_path_walk,
    simple_graph_path_vertices_unique
from graph.simple_graph_degree import degree
from graph.simple_graph_edges import directed_edges

numerals Nat

/// True when `steps` traces a cycle in `g`: a closed walk of length at least three
/// whose vertices are all distinct except for the repeated start vertex.
///
/// The distinctness is stated on the target list `steps`. Its last entry is the start
/// vertex, so uniqueness of `steps` means exactly that the internal vertices are distinct
/// and different from the start.
define simple_graph_cycle[V](g: SimpleGraph[V], start: V, steps: List[V]) -> Bool {
    simple_graph_closed_walk(g, start, steps) and Nat.3 <= steps.length and steps.is_unique
}

/// The graph is acyclic on the vertex set `s`: no cycle of the graph has all of its
/// vertices in `s`.
define is_acyclic[V](g: SimpleGraph[V], s: FiniteSet[V]) -> Bool {
    forall(start: V, steps: List[V]) {
        not (s.contains(start) and simple_graph_cycle(g, start, steps) and
            (forall(x: V) { steps.contains(x) implies s.contains(x) }))
    }
}

/// The graph is connected on the vertex set `s`: every two vertices of `s` are joined by
/// a walk that stays inside `s`.
define simple_graph_connected_on[V](g: SimpleGraph[V], s: FiniteSet[V]) -> Bool {
    forall(x: V, y: V) {
        s.contains(x) and s.contains(y) implies
            simple_graph_reachable(induced_subgraph(g, s.underlying_set), x, y)
    }
}

/// A tree on the vertex set `s`: a connected and acyclic graph whose vertices are `s`.
define is_tree[V](g: SimpleGraph[V], s: FiniteSet[V]) -> Bool {
    simple_graph_connected_on(g, s) and is_acyclic(g, s)
}

/// A leaf of `g` in `s`: a vertex of `s` adjacent to exactly one other vertex of `s`.
define is_leaf[V](g: SimpleGraph[V], s: FiniteSet[V], v: V) -> Bool {
    exists(u: V) {
        s.contains(u) and g.adj(v, u) and
            forall(w: V) {
                s.contains(w) and g.adj(v, w) implies w = u
            }
    }
}

/// Acyclicity is inherited by smaller vertex sets: a cycle lying in `t` also lies in `s`.
theorem is_acyclic_subset[V](g: SimpleGraph[V], s: FiniteSet[V], t: FiniteSet[V]) {
    is_acyclic(g, s) and t.subset_eq(s) implies is_acyclic(g, t)
} by {
    if is_acyclic(g, s) and t.subset_eq(s) {
        is_acyclic(g, s) = forall(start: V, steps: List[V]) {
            not (s.contains(start) and simple_graph_cycle(g, start, steps) and
                (forall(x: V) { steps.contains(x) implies s.contains(x) }))
        }
        forall(start: V, steps: List[V]) {
            if t.contains(start) and simple_graph_cycle(g, start, steps) and
                (forall(x: V) { steps.contains(x) implies t.contains(x) }) {
                finite_set_subset_contains(t, s, start)
                t.contains(start) implies s.contains(start)
                s.contains(start)
                forall(x: V) {
                    if steps.contains(x) {
                        steps.contains(x) implies t.contains(x)
                        t.contains(x)
                        finite_set_subset_contains(t, s, x)
                        t.contains(x) implies s.contains(x)
                        s.contains(x)
                    }
                    steps.contains(x) implies s.contains(x)
                }
                not (s.contains(start) and simple_graph_cycle(g, start, steps) and
                    (forall(x: V) { steps.contains(x) implies s.contains(x) }))
                s.contains(start) and simple_graph_cycle(g, start, steps) and
                    (forall(x: V) { steps.contains(x) implies s.contains(x) })
                false
            }
            not (t.contains(start) and simple_graph_cycle(g, start, steps) and
                (forall(x: V) { steps.contains(x) implies t.contains(x) }))
        }
        is_acyclic(g, t) = forall(start: V, steps: List[V]) {
            not (t.contains(start) and simple_graph_cycle(g, start, steps) and
                (forall(x: V) { steps.contains(x) implies t.contains(x) }))
        }
        is_acyclic(g, t)
    }
}

/// A list of length at least three is a cons of a cons of a cons.
theorem list_three_of_length[T](steps: List[T]) {
    Nat.3 <= steps.length implies exists(a: T, b: T, c: T, tail: List[T]) {
        steps = List.cons(a, List.cons(b, List.cons(c, tail)))
    }
} by {
    if Nat.3 <= steps.length {
        match steps {
            List.nil {
                List.nil[T].length = Nat.0
                Nat.3 <= List.nil[T].length
                Nat.3 <= Nat.0
                only_zero_lte_zero(Nat.3)
                Nat.3 = Nat.0
                false
            }
            List.cons(w1, rest) {
                List.cons(w1, rest).length = rest.length + Nat.1
                match rest {
                    List.nil {
                        List.nil[T].length = Nat.0
                        Nat.3 <= List.cons(w1, List.nil[T]).length
                        Nat.3 <= Nat.0 + Nat.1
                        Nat.3 <= Nat.1
                        lt_suc(Nat.1)
                        Nat.1 < Nat.2
                        lt_suc(Nat.2)
                        Nat.2 < Nat.3
                        lt_trans(Nat.1, Nat.2, Nat.3)
                        Nat.1 < Nat.3
                        lte_and_lt(Nat.3, Nat.1, Nat.3)
                        Nat.3 < Nat.3
                        false
                    }
                    List.cons(w2, rest2) {
                        List.cons(w2, rest2).length = rest2.length + Nat.1
                        match rest2 {
                            List.nil {
                                List.nil[T].length = Nat.0
                                Nat.3 <= List.cons(w1, List.cons(w2, List.nil[T])).length
                                Nat.3 <= Nat.0 + Nat.1 + Nat.1
                                Nat.3 <= Nat.2
                                lt_suc(Nat.2)
                                Nat.2 < Nat.3
                                lte_and_lt(Nat.3, Nat.2, Nat.3)
                                Nat.3 < Nat.3
                                false
                            }
                            List.cons(w3, rest3) {
                                List.cons(w3, rest3).length = rest3.length + Nat.1
                                steps = List.cons(w1, rest)
                                List.cons(w1, rest) = List.cons(w1, List.cons(w2, rest2))
                                List.cons(w1, List.cons(w2, rest2)) =
                                    List.cons(w1, List.cons(w2, List.cons(w3, rest3)))
                                steps = List.cons(w1, List.cons(w2, List.cons(w3, rest3)))
                                exists(a: T, b: T, c: T, tail: List[T]) {
                                    steps = List.cons(a, List.cons(b, List.cons(c, tail)))
                                }
                            }
                        }
                    }
                }
            }
        }
    }
}

/// A cycle of length at least three carries three pairwise distinct vertices of `s`.
///
/// Read off the first three targets of the walk: uniqueness of `steps` makes them pairwise
/// distinct, and the membership hypothesis puts all of them in `s`.
theorem cycle_has_three_distinct[V](g: SimpleGraph[V], s: FiniteSet[V], start: V, steps: List[V]) {
    simple_graph_cycle(g, start, steps) and
        (forall(x: V) { steps.contains(x) implies s.contains(x) })
        implies exists(w1: V, w2: V, w3: V) {
            s.contains(w1) and s.contains(w2) and s.contains(w3) and
            w1 != w2 and w1 != w3 and w2 != w3
        }
} by {
    if simple_graph_cycle(g, start, steps) and
        (forall(x: V) { steps.contains(x) implies s.contains(x) }) {
        simple_graph_cycle(g, start, steps) = (simple_graph_closed_walk(g, start, steps) and
            Nat.3 <= steps.length and steps.is_unique)
        Nat.3 <= steps.length
        steps.is_unique
        list_three_of_length(steps)
        exists(w1: V, w2: V, w3: V, rest: List[V]) {
            steps = List.cons(w1, List.cons(w2, List.cons(w3, rest)))
        }
        let (w1: V, w2: V, w3: V, rest: List[V]) satisfy {
            steps = List.cons(w1, List.cons(w2, List.cons(w3, rest)))
        }
        List.cons(w1, List.cons(w2, List.cons(w3, rest))).is_unique
        unique_cons_head_fresh(w1, List.cons(w2, List.cons(w3, rest)), w2)
        w1 != w2
        unique_cons_head_fresh(w1, List.cons(w2, List.cons(w3, rest)), w3)
        w1 != w3
        unique_cons_head_fresh(w2, List.cons(w3, rest), w3)
        w2 != w3
        List.cons(w1, List.cons(w2, List.cons(w3, rest))).contains(w1)
        steps.contains(w1)
        steps.contains(w1) implies s.contains(w1)
        s.contains(w1)
        List.cons(w2, List.cons(w3, rest)).contains(w2)
        List.cons(w1, List.cons(w2, List.cons(w3, rest))).contains(w2)
        steps.contains(w2)
        steps.contains(w2) implies s.contains(w2)
        s.contains(w2)
        List.cons(w3, rest).contains(w3)
        List.cons(w2, List.cons(w3, rest)).contains(w3)
        List.cons(w1, List.cons(w2, List.cons(w3, rest))).contains(w3)
        steps.contains(w3)
        steps.contains(w3) implies s.contains(w3)
        s.contains(w3)
        exists(a: V, b: V, c: V) {
            s.contains(a) and s.contains(b) and s.contains(c) and
            a != b and a != c and b != c
        }
    }
}

/// A finite set with three pairwise distinct members has cardinality at least three.
theorem fs_card_three_of_distinct_members[T](s: FiniteSet[T], w1: T, w2: T, w3: T) {
    s.contains(w1) and s.contains(w2) and s.contains(w3) and
    w1 != w2 and w1 != w3 and w2 != w3 implies Nat.3 <= fs_card(s)
} by {
    if s.contains(w1) and s.contains(w2) and s.contains(w3) and
        w1 != w2 and w1 != w3 and w2 != w3 {
        finite_set_empty_contains_eq(w1)
        not FiniteSet.empty[T].contains(w1)
        fs_card_insert_of_not_contains(FiniteSet.empty[T], w1)
        fs_card(fs_insert(FiniteSet.empty[T], w1)) = fs_card(FiniteSet.empty[T]) + Nat.1
        fs_card_empty[T]
        fs_card(FiniteSet.empty[T]) = Nat.0
        fs_card(fs_insert(FiniteSet.empty[T], w1)) = Nat.1
        fs_insert_contains_eq(FiniteSet.empty[T], w1, w2)
        (fs_insert(FiniteSet.empty[T], w1).contains(w2) = (w2 = w1 or FiniteSet.empty[T].contains(w2)))
        finite_set_empty_contains_eq(w2)
        if w2 = w1 {
            w1 != w2
            false
        }
        not fs_insert(FiniteSet.empty[T], w1).contains(w2)
        fs_card_insert_of_not_contains(fs_insert(FiniteSet.empty[T], w1), w2)
        fs_card(fs_insert(fs_insert(FiniteSet.empty[T], w1), w2)) =
            fs_card(fs_insert(FiniteSet.empty[T], w1)) + Nat.1
        fs_card(fs_insert(fs_insert(FiniteSet.empty[T], w1), w2)) = Nat.2
        fs_insert_contains_eq(fs_insert(FiniteSet.empty[T], w1), w2, w3)
        (fs_insert(fs_insert(FiniteSet.empty[T], w1), w2).contains(w3)
            = (w3 = w2 or fs_insert(FiniteSet.empty[T], w1).contains(w3)))
        fs_insert_contains_eq(FiniteSet.empty[T], w1, w3)
        (fs_insert(FiniteSet.empty[T], w1).contains(w3) = (w3 = w1 or FiniteSet.empty[T].contains(w3)))
        finite_set_empty_contains_eq(w3)
        if w3 = w1 {
            w1 != w3
            false
        }
        if w3 = w2 {
            w2 != w3
            false
        }
        not fs_insert(fs_insert(FiniteSet.empty[T], w1), w2).contains(w3)
        fs_card_insert_of_not_contains(fs_insert(fs_insert(FiniteSet.empty[T], w1), w2), w3)
        fs_card(fs_insert(fs_insert(fs_insert(FiniteSet.empty[T], w1), w2), w3)) =
            fs_card(fs_insert(fs_insert(FiniteSet.empty[T], w1), w2)) + Nat.1
        fs_card(fs_insert(fs_insert(fs_insert(FiniteSet.empty[T], w1), w2), w3)) = Nat.3
        forall(z: T) {
            if fs_insert(fs_insert(fs_insert(FiniteSet.empty[T], w1), w2), w3).contains(z) {
                fs_insert_contains_eq(fs_insert(fs_insert(FiniteSet.empty[T], w1), w2), w3, z)
                (fs_insert(fs_insert(fs_insert(FiniteSet.empty[T], w1), w2), w3).contains(z)
                    = (z = w3 or fs_insert(fs_insert(FiniteSet.empty[T], w1), w2).contains(z)))
                if z = w3 {
                    s.contains(z)
                }
                if z != w3 {
                    fs_insert(fs_insert(FiniteSet.empty[T], w1), w2).contains(z)
                    fs_insert_contains_eq(fs_insert(FiniteSet.empty[T], w1), w2, z)
                    (fs_insert(fs_insert(FiniteSet.empty[T], w1), w2).contains(z)
                        = (z = w2 or fs_insert(FiniteSet.empty[T], w1).contains(z)))
                    if z = w2 {
                        s.contains(z)
                    }
                    if z != w2 {
                        fs_insert(FiniteSet.empty[T], w1).contains(z)
                        fs_insert_contains_eq(FiniteSet.empty[T], w1, z)
                        (fs_insert(FiniteSet.empty[T], w1).contains(z)
                            = (z = w1 or FiniteSet.empty[T].contains(z)))
                        finite_set_empty_contains_eq(z)
                        z = w1
                        s.contains(z)
                    }
                    s.contains(z)
                }
                s.contains(z)
            }
            (fs_insert(fs_insert(fs_insert(FiniteSet.empty[T], w1), w2), w3).contains(z) implies s.contains(z))
        }
        fs_subset_eq_intro(fs_insert(fs_insert(fs_insert(FiniteSet.empty[T], w1), w2), w3), s)
        fs_insert(fs_insert(fs_insert(FiniteSet.empty[T], w1), w2), w3).subset_eq(s)
        fs_card_mono(fs_insert(fs_insert(fs_insert(FiniteSet.empty[T], w1), w2), w3), s)
        fs_card(fs_insert(fs_insert(fs_insert(FiniteSet.empty[T], w1), w2), w3)) <= fs_card(s)
        Nat.3 <= fs_card(s)
    }
}

/// A graph is acyclic on any vertex set of at most two vertices: a cycle of length at
/// least three needs three distinct vertices.
theorem is_acyclic_of_card_le_two[V](g: SimpleGraph[V], s: FiniteSet[V]) {
    fs_card(s) <= Nat.2 implies is_acyclic(g, s)
} by {
    if fs_card(s) <= Nat.2 {
        forall(start: V, steps: List[V]) {
            if s.contains(start) and simple_graph_cycle(g, start, steps) and
                (forall(x: V) { steps.contains(x) implies s.contains(x) }) {
                cycle_has_three_distinct(g, s, start, steps)
                exists(w1: V, w2: V, w3: V) {
                    s.contains(w1) and s.contains(w2) and s.contains(w3) and
                    w1 != w2 and w1 != w3 and w2 != w3
                }
                let (w1: V, w2: V, w3: V) satisfy {
                    s.contains(w1) and s.contains(w2) and s.contains(w3) and
                    w1 != w2 and w1 != w3 and w2 != w3
                }
                fs_card_three_of_distinct_members(s, w1, w2, w3)
                Nat.3 <= fs_card(s)
                lte_trans(Nat.3, fs_card(s), Nat.2)
                Nat.3 <= Nat.2
                Nat.2 < Nat.3
                lte_and_lt(Nat.3, Nat.2, Nat.3)
                Nat.3 < Nat.3
                false
            }
            not (s.contains(start) and simple_graph_cycle(g, start, steps) and
                (forall(x: V) { steps.contains(x) implies s.contains(x) }))
        }
        is_acyclic(g, s) = forall(start: V, steps: List[V]) {
            not (s.contains(start) and simple_graph_cycle(g, start, steps) and
                (forall(x: V) { steps.contains(x) implies s.contains(x) }))
        }
        is_acyclic(g, s)
    }
}

/// A tree is connected on its vertex set.
theorem tree_connected_on[V](g: SimpleGraph[V], s: FiniteSet[V]) {
    is_tree(g, s) implies simple_graph_connected_on(g, s)
} by {
    if is_tree(g, s) {
        is_tree(g, s) = (simple_graph_connected_on(g, s) and is_acyclic(g, s))
        simple_graph_connected_on(g, s)
    }
}

/// A tree is acyclic on its vertex set.
theorem tree_acyclic[V](g: SimpleGraph[V], s: FiniteSet[V]) {
    is_tree(g, s) implies is_acyclic(g, s)
} by {
    if is_tree(g, s) {
        is_tree(g, s) = (simple_graph_connected_on(g, s) and is_acyclic(g, s))
        is_acyclic(g, s)
    }
}

/// A tree with at least two vertices has an edge.
///
/// Two distinct vertices of `s` are joined by a walk that stays inside `s`, and the first
/// step of that walk is an edge.
theorem tree_has_edge[V](g: SimpleGraph[V], s: FiniteSet[V]) {
    is_tree(g, s) and Nat.2 <= fs_card(s) implies
        exists(x: V, y: V) {
            s.contains(x) and s.contains(y) and x != y and g.adj(x, y)
        }
} by {
    if is_tree(g, s) and Nat.2 <= fs_card(s) {
        tree_connected_on(g, s)
        simple_graph_connected_on(g, s)
        simple_graph_connected_on(g, s) = forall(x: V, y: V) {
            s.contains(x) and s.contains(y) implies
                simple_graph_reachable(induced_subgraph(g, s.underlying_set), x, y)
        }
        fs_two_distinct_members(s)
        exists(a: V, b: V) {
            s.contains(a) and s.contains(b) and a != b
        }
        let (a: V, b: V) satisfy {
            s.contains(a) and s.contains(b) and a != b
        }
        simple_graph_reachable(induced_subgraph(g, s.underlying_set), a, b)
        simple_graph_reachable_has_walk(induced_subgraph(g, s.underlying_set), a, b)
        exists(steps: List[V]) {
            simple_graph_walk(induced_subgraph(g, s.underlying_set), a, b, steps)
        }
        let steps: List[V] satisfy {
            simple_graph_walk(induced_subgraph(g, s.underlying_set), a, b, steps)
        }
        match steps {
            List.nil {
                simple_graph_walk_nil(induced_subgraph(g, s.underlying_set), a, b)
                simple_graph_walk(induced_subgraph(g, s.underlying_set), a, b, List.nil[V]) = (a = b)
                a = b
                a != b
                false
            }
            List.cons(w, rest) {
                simple_graph_walk_cons_iff(induced_subgraph(g, s.underlying_set), a, b, w, rest)
                simple_graph_walk(induced_subgraph(g, s.underlying_set), a, b, List.cons(w, rest)) =
                    (induced_subgraph(g, s.underlying_set).adj(a, w) and
                        simple_graph_walk(induced_subgraph(g, s.underlying_set), w, b, rest))
                induced_subgraph(g, s.underlying_set).adj(a, w)
                induced_subgraph_adj_eq(g, s.underlying_set, a, w)
                induced_subgraph(g, s.underlying_set).adj(a, w) =
                    (g.adj(a, w) and s.contains(a) and s.contains(w))
                g.adj(a, w)
                s.contains(a)
                s.contains(w)
                simple_graph_adj_ne(g, a, w)
                g.adj(a, w) implies a != w
                a != w
                exists(x: V, y: V) {
                    s.contains(x) and s.contains(y) and x != y and g.adj(x, y)
                }
            }
        }
    }
}

/// True of the lengths achieved by paths of the subgraph of `g` induced by `s`.
/// True of the lengths achieved by paths of `g` whose vertices all lie in `s`.
define path_length_pred[V](g: SimpleGraph[V], s: FiniteSet[V]) -> (Nat -> Bool) {
    function(n: Nat) {
        exists(start: V, finish: V, steps: List[V]) {
            simple_graph_path(g, start, finish, steps) and
            s.contains(start) and
            (forall(x: V) { steps.contains(x) implies s.contains(x) }) and
            steps.length = n
        }
    }
}

/// Zero is an achieved path length: the trivial path stays at its start vertex.
theorem path_length_pred_zero_at[V](g: SimpleGraph[V], s: FiniteSet[V], start: V) {
    s.contains(start) implies path_length_pred(g, s)(Nat.0)
} by {
    if s.contains(start) {
        path_length_pred(g, s)(Nat.0) = exists(a: V, b: V, steps: List[V]) {
            simple_graph_path(g, a, b, steps) and
            s.contains(a) and
            (forall(x: V) { steps.contains(x) implies s.contains(x) }) and
            steps.length = Nat.0
        }
        simple_graph_walk_refl(g, start)
        simple_graph_walk(g, start, start, List.nil[V])
        simple_graph_walk_vertices(start, List.nil[V]) = List.cons(start, List.nil[V])
        List.singleton(start) = List.cons(start, List.nil[V])
        simple_graph_walk_vertices(start, List.nil[V]) = List.singleton(start)
        singleton_unique(start)
        List.singleton(start).is_unique
        simple_graph_walk_vertices(start, List.nil[V]).is_unique
        simple_graph_path(g, start, start, List.nil[V])
        List.nil[V].length = Nat.0
        (forall(x: V) { List.nil[V].contains(x) implies s.contains(x) }) = true
        eq_true_elim(forall(x: V) { List.nil[V].contains(x) implies s.contains(x) })
        forall(x: V) { List.nil[V].contains(x) implies s.contains(x) }
        exists(a: V, b: V, steps: List[V]) {
            simple_graph_path(g, a, b, steps) and
            s.contains(a) and
            (forall(x: V) { steps.contains(x) implies s.contains(x) }) and
            steps.length = Nat.0
        }
        path_length_pred(g, s)(Nat.0)
    }
}

/// Prepending a fresh element to a unique list keeps it unique.
theorem unique_cons_intro[T](head: T, tail: List[T]) {
    not tail.contains(head) and tail.is_unique implies List.cons(head, tail).is_unique
} by {
    if not tail.contains(head) and tail.is_unique {
        List.cons(head, tail).unique = List.cons(head, tail.unique)
        tail.unique = tail
        List.cons(head, tail.unique) = List.cons(head, tail)
        List.cons(head, tail).unique = List.cons(head, tail)
        List.cons(head, tail).is_unique
    }
}

/// An edge of `s` gives a path of length one.
theorem path_length_pred_of_edge[V](g: SimpleGraph[V], s: FiniteSet[V], x: V, y: V) {
    s.contains(x) and s.contains(y) and g.adj(x, y) implies path_length_pred(g, s)(Nat.1)
} by {
    if s.contains(x) and s.contains(y) and g.adj(x, y) {
        simple_graph_walk_of_adj(g, x, y)
        simple_graph_walk(g, x, y, List.singleton(y))
        simple_graph_adj_ne(g, x, y)
        g.adj(x, y) implies x != y
        x != y
        simple_graph_walk_vertices(x, List.singleton(y)) = List.cons(x, List.singleton(y))
        List.singleton(y) = List.cons(y, List.nil[V])
        simple_graph_walk_vertices(x, List.singleton(y)) = List.cons(x, List.cons(y, List.nil[V]))
        if List.singleton(y).contains(x) {
            singleton_contains_imp_eq(y, x)
            List.singleton(y).contains(x) implies x = y
            x = y
            x != y
            false
        }
        not List.singleton(y).contains(x)
        singleton_unique(y)
        List.singleton(y).is_unique
        unique_cons_intro(x, List.singleton(y))
        List.cons(x, List.singleton(y)).is_unique
        List.cons(x, List.cons(y, List.nil[V])).is_unique
        simple_graph_walk_vertices(x, List.singleton(y)).is_unique
        simple_graph_path(g, x, y, List.singleton(y))
        List.cons(y, List.nil[V]).length = List.nil[V].length + Nat.1
        List.nil[V].length = Nat.0
        List.cons(y, List.nil[V]).length = Nat.0 + Nat.1
        List.cons(y, List.nil[V]).length = Nat.1
        List.singleton(y) = List.cons(y, List.nil[V])
        List.singleton(y).length = Nat.1
        forall(z: V) {
            if List.singleton(y).contains(z) {
                singleton_contains_imp_eq(y, z)
                List.singleton(y).contains(z) implies z = y
                z = y
                s.contains(y)
                z = y
                s.contains(z)
            }
            List.singleton(y).contains(z) implies s.contains(z)
        }
        path_length_pred(g, s)(Nat.1) = exists(a: V, b: V, steps: List[V]) {
            simple_graph_path(g, a, b, steps) and
            s.contains(a) and
            (forall(u: V) { steps.contains(u) implies s.contains(u) }) and
            steps.length = Nat.1
        }
        exists(a: V, b: V, steps: List[V]) {
            simple_graph_path(g, a, b, steps) and
            s.contains(a) and
            (forall(u: V) { steps.contains(u) implies s.contains(u) }) and
            steps.length = Nat.1
        }
        path_length_pred(g, s)(Nat.1)
    }
}

/// Path lengths are bounded by the number of vertices of `s`.
theorem path_length_pred_bounded[V](g: SimpleGraph[V], s: FiniteSet[V]) {
    is_upper_bound_of(path_length_pred(g, s), fs_card(s))
} by {
    forall(n: Nat) {
        if path_length_pred(g, s)(n) {
            path_length_pred(g, s)(n) = exists(a: V, b: V, steps: List[V]) {
                simple_graph_path(g, a, b, steps) and
                s.contains(a) and
                (forall(x: V) { steps.contains(x) implies s.contains(x) }) and
                steps.length = n
            }
            let (a: V, b: V, steps: List[V]) satisfy {
                simple_graph_path(g, a, b, steps) and
                s.contains(a) and
                (forall(x: V) { steps.contains(x) implies s.contains(x) }) and
                steps.length = n
            }
            if steps.length = Nat.0 {
                steps.length = n
                n = Nat.0
                Nat.0 <= fs_card(s)
                n <= fs_card(s)
            }
            if not (steps.length = Nat.0) {
                steps.length != Nat.0
                pos_of_ne_zero(steps.length)
                Nat.0 < steps.length
                lt_imp_lte_suc(Nat.0, steps.length)
                Nat.1 <= steps.length
                simple_graph_path_vertices_unique(g, a, b, steps)
                simple_graph_walk_vertices(a, steps).is_unique
                simple_graph_walk_vertices(a, steps) = List.cons(a, steps)
                List.cons(a, steps).is_unique
                forall(x: V) {
                    if List.cons(a, steps).contains(x) {
                        if x = a {
                            s.contains(a)
                            x = a
                            s.contains(x)
                        }
                        if x != a {
                            List.cons(a, steps).contains(x) = (a = x or steps.contains(x))
                            steps.contains(x)
                            steps.contains(x) implies s.contains(x)
                            s.contains(x)
                        }
                        s.contains(x)
                    }
                    List.cons(a, steps).contains(x) implies s.contains(x)
                }
                finite_set_from_unique_list_cardinality_is_length(List.cons(a, steps))
                fs_from_list(List.cons(a, steps)).cardinality_is(List.cons(a, steps).length)
                fs_card_eq_of_cardinality_is(fs_from_list(List.cons(a, steps)), List.cons(a, steps).length)
                fs_card(fs_from_list(List.cons(a, steps))) = List.cons(a, steps).length
                forall(x: V) {
                    if fs_from_list(List.cons(a, steps)).contains(x) {
                        fs_from_list_contains_eq(List.cons(a, steps), x)
                        fs_from_list(List.cons(a, steps)).contains(x) = List.cons(a, steps).contains(x)
                        List.cons(a, steps).contains(x) implies s.contains(x)
                        s.contains(x)
                    }
                    fs_from_list(List.cons(a, steps)).contains(x) implies s.contains(x)
                }
                fs_subset_eq_intro(fs_from_list(List.cons(a, steps)), s)
                fs_from_list(List.cons(a, steps)).subset_eq(s)
                fs_card_mono(fs_from_list(List.cons(a, steps)), s)
                fs_card(fs_from_list(List.cons(a, steps))) <= fs_card(s)
                List.cons(a, steps).length = steps.length + Nat.1
                fs_card(fs_from_list(List.cons(a, steps))) = steps.length + Nat.1
                steps.length + Nat.1 <= fs_card(s)
                steps.length = n
                n + Nat.1 <= fs_card(s)
                n <= n.suc
                add_one_right(n)
                n + Nat.1 = n.suc
                n <= n + Nat.1
                lte_trans(n, n + Nat.1, fs_card(s))
                n <= fs_card(s)
            }
            n <= fs_card(s)
        }
        path_length_pred(g, s)(n) implies n <= fs_card(s)
    }
    is_upper_bound_of_intro(path_length_pred(g, s), fs_card(s))
    is_upper_bound_of(path_length_pred(g, s), fs_card(s))
}

/// A tree with at least two vertices has a path of maximal length.
///
/// The edge of the tree gives a path of length one, and every path length is bounded by
/// the number of vertices, so a greatest achieved length exists.
theorem maximum_path_length_exists[V](g: SimpleGraph[V], s: FiniteSet[V]) {
    is_tree(g, s) and Nat.2 <= fs_card(s) implies
        exists(m: Nat) {
            is_max(path_length_pred(g, s), m)
        }
} by {
    if is_tree(g, s) and Nat.2 <= fs_card(s) {
        tree_has_edge(g, s)
        exists(x: V, y: V) {
            s.contains(x) and s.contains(y) and x != y and g.adj(x, y)
        }
        let (x: V, y: V) satisfy {
            s.contains(x) and s.contains(y) and x != y and g.adj(x, y)
        }
        path_length_pred_of_edge(g, s, x, y)
        path_length_pred(g, s)(Nat.1)
        path_length_pred_bounded(g, s)
        has_max(path_length_pred(g, s), Nat.1, fs_card(s))
        exists(m: Nat) {
            is_max(path_length_pred(g, s), m)
        }
    }
}


/// Every vertex of a connected graph on `s` with at least two vertices has a neighbor in `s`.
///
/// Pick a second vertex, join it to `v` by a walk inside `s`, and take the first step.
theorem connected_on_vertex_has_neighbor[V](g: SimpleGraph[V], s: FiniteSet[V], v: V) {
    simple_graph_connected_on(g, s) and s.contains(v) and Nat.2 <= fs_card(s) implies
        exists(u: V) { s.contains(u) and g.adj(v, u) }
} by {
    if simple_graph_connected_on(g, s) and s.contains(v) and Nat.2 <= fs_card(s) {
        fs_card_remove_of_contains(s, v)
        fs_card(s) = fs_card(fs_remove(s, v)) + Nat.1
        add_one_right(fs_card(fs_remove(s, v)))
        fs_card(fs_remove(s, v)) + Nat.1 = fs_card(fs_remove(s, v)).suc
        fs_card(s) = fs_card(fs_remove(s, v)).suc
        lte_cancel_suc(Nat.1, fs_card(fs_remove(s, v)))
        Nat.1.suc <= fs_card(fs_remove(s, v)).suc implies Nat.1 <= fs_card(fs_remove(s, v))
        Nat.1.suc <= fs_card(s)
        Nat.2 <= fs_card(s)
        Nat.1.suc = Nat.2
        Nat.1 <= fs_card(fs_remove(s, v))
        fs_member_of_card_pos(fs_remove(s, v))
        exists(x: V) { fs_remove(s, v).contains(x) }
        let (u: V) satisfy {
            fs_remove(s, v).contains(u)
        }
        fs_remove_contains_eq(s, v, u)
        fs_remove(s, v).contains(u) = (s.contains(u) and u != v)
        s.contains(u)
        u != v
        simple_graph_connected_on(g, s) = forall(x: V, y: V) {
            s.contains(x) and s.contains(y) implies
                simple_graph_reachable(induced_subgraph(g, s.underlying_set), x, y)
        }
        simple_graph_reachable(induced_subgraph(g, s.underlying_set), v, u)
        simple_graph_reachable_has_walk(induced_subgraph(g, s.underlying_set), v, u)
        exists(steps: List[V]) {
            simple_graph_walk(induced_subgraph(g, s.underlying_set), v, u, steps)
        }
        let steps: List[V] satisfy {
            simple_graph_walk(induced_subgraph(g, s.underlying_set), v, u, steps)
        }
        match steps {
            List.nil {
                simple_graph_walk_nil(induced_subgraph(g, s.underlying_set), v, u)
                simple_graph_walk(induced_subgraph(g, s.underlying_set), v, u, List.nil[V]) = (v = u)
                v = u
                u != v
                false
            }
            List.cons(w, rest) {
                simple_graph_walk_cons_iff(induced_subgraph(g, s.underlying_set), v, u, w, rest)
                simple_graph_walk(induced_subgraph(g, s.underlying_set), v, u, List.cons(w, rest)) =
                    (induced_subgraph(g, s.underlying_set).adj(v, w) and
                        simple_graph_walk(induced_subgraph(g, s.underlying_set), w, u, rest))
                induced_subgraph(g, s.underlying_set).adj(v, w)
                induced_subgraph_adj_eq(g, s.underlying_set, v, w)
                induced_subgraph(g, s.underlying_set).adj(v, w) =
                    (g.adj(v, w) and s.underlying_set.contains(v) and s.underlying_set.contains(w))
                g.adj(v, w)
                s.contains(w)
                exists(z: V) {
                    s.contains(z) and g.adj(v, z)
                }
            }
        }
    }
}

/// A walk splits at its start vertex into an empty prefix and the full walk.
theorem walk_split_at_start[V](g: SimpleGraph[V], a: V, c: V, b: V, steps: List[V]) {
    simple_graph_walk(g, a, b, steps) and a = c implies
        exists(p: List[V], s: List[V]) {
            steps = p + s and simple_graph_walk(g, a, c, p) and simple_graph_walk(g, c, b, s)
        }
} by {
    if simple_graph_walk(g, a, b, steps) and a = c {
        simple_graph_walk_refl(g, a)
        simple_graph_walk(g, a, a, List.nil[V])
        a = c
        simple_graph_walk(g, a, c, List.nil[V])
        simple_graph_walk(g, c, b, steps)
        List.nil[V] + steps = steps
        exists(pre: List[V], suf: List[V]) {
            steps = pre + suf and simple_graph_walk(g, a, c, pre) and simple_graph_walk(g, c, b, suf)
        }
    }
}

/// A walk splits at any vertex that appears among its targets.
///
/// Walking down the target list, the first occurrence of `c` is the split point: the
/// prefix ends at `c` and the suffix continues to the finish.
theorem walk_split_in_steps[V](g: SimpleGraph[V], a0: V, c0: V, b0: V, steps: List[V]) {
    simple_graph_walk(g, a0, b0, steps) and steps.contains(c0) implies
        exists(pre: List[V], suf: List[V]) {
            steps = pre + suf and simple_graph_walk(g, a0, c0, pre) and simple_graph_walk(g, c0, b0, suf)
        }
} by {
    define pc(xs: List[V], a: V, c: V, b: V) -> Bool {
        simple_graph_walk(g, a, b, xs) and xs.contains(c) implies
            exists(pre: List[V], suf: List[V]) {
                xs = pre + suf and simple_graph_walk(g, a, c, pre) and simple_graph_walk(g, c, b, suf)
            }
    }

    define p(xs: List[V]) -> Bool {
        forall(a: V, c: V, b: V) {
            pc(xs, a, c, b)
        }
    }

    forall(a: V, c: V, b: V) {
        (simple_graph_walk(g, a, b, List.nil[V]) and List.nil[V].contains(c) implies
            exists(pre: List[V], suf: List[V]) {
                List.nil[V] = pre + suf and simple_graph_walk(g, a, c, pre) and
                    simple_graph_walk(g, c, b, suf)
            }) = true
        eq_true_elim(simple_graph_walk(g, a, b, List.nil[V]) and List.nil[V].contains(c) implies
            exists(pre: List[V], suf: List[V]) {
                List.nil[V] = pre + suf and simple_graph_walk(g, a, c, pre) and
                    simple_graph_walk(g, c, b, suf)
            })
        (simple_graph_walk(g, a, b, List.nil[V]) and List.nil[V].contains(c) implies
            exists(pre: List[V], suf: List[V]) {
                List.nil[V] = pre + suf and simple_graph_walk(g, a, c, pre) and
                    simple_graph_walk(g, c, b, suf)
            })
        pc(List.nil[V], a, c, b)
    }
    p(List.nil[V])

    forall(head: V, tail: List[V]) {
        if p(tail) {
            forall(a: V, c: V, b: V) {
                if simple_graph_walk(g, a, b, List.cons(head, tail)) and
                    List.cons(head, tail).contains(c) {
                    simple_graph_walk_cons_iff(g, a, b, head, tail)
                    simple_graph_walk(g, a, b, List.cons(head, tail)) =
                        (g.adj(a, head) and simple_graph_walk(g, head, b, tail))
                    simple_graph_walk(g, head, b, tail)
                    if head = c {
                        List.cons(head, tail) = List.cons(head, List.nil[V]) + tail
                        simple_graph_walk_cons_iff(g, a, c, head, List.nil[V])
                        simple_graph_walk(g, a, c, List.cons(head, List.nil[V])) =
                            (g.adj(a, head) and simple_graph_walk(g, head, c, List.nil[V]))
                        simple_graph_walk_nil(g, head, c)
                        simple_graph_walk(g, head, c, List.nil[V]) = (head = c)
                        simple_graph_walk(g, a, c, List.cons(head, List.nil[V]))
                        exists(pre: List[V], suf: List[V]) {
                            List.cons(head, tail) = pre + suf and
                                simple_graph_walk(g, a, c, pre) and simple_graph_walk(g, c, b, suf)
                        }
                    }
                    if head != c {
                        List.cons(head, tail).contains(c)
                        List.cons(head, tail).contains(c) = (head = c or tail.contains(c))
                        tail.contains(c)
                        pc(tail, head, c, b)
                        pc(tail, head, c, b) = (simple_graph_walk(g, head, b, tail) and tail.contains(c) implies
                            exists(pre: List[V], suf: List[V]) {
                                tail = pre + suf and simple_graph_walk(g, head, c, pre) and
                                    simple_graph_walk(g, c, b, suf)
                            })
                        exists(pre: List[V], suf: List[V]) {
                            tail = pre + suf and simple_graph_walk(g, head, c, pre) and
                                simple_graph_walk(g, c, b, suf)
                        }
                        let (pre: List[V], suf: List[V]) satisfy {
                            tail = pre + suf and simple_graph_walk(g, head, c, pre) and
                                simple_graph_walk(g, c, b, suf)
                        }
                        List.cons(head, tail) = List.cons(head, pre) + suf
                        List.cons(head, pre) + suf = List.cons(head, pre + suf)
                        List.cons(head, tail) = List.cons(head, pre + suf)
                        tail = pre + suf
                        simple_graph_walk_cons_iff(g, a, c, head, pre)
                        simple_graph_walk(g, a, c, List.cons(head, pre)) =
                            (g.adj(a, head) and simple_graph_walk(g, head, c, pre))
                        g.adj(a, head)
                        simple_graph_walk(g, a, c, List.cons(head, pre))
                        exists(x: List[V], y: List[V]) {
                            List.cons(head, tail) = x + y and
                                simple_graph_walk(g, a, c, x) and simple_graph_walk(g, c, b, y)
                        }
                    }
                    exists(x: List[V], y: List[V]) {
                        List.cons(head, tail) = x + y and
                            simple_graph_walk(g, a, c, x) and simple_graph_walk(g, c, b, y)
                    }
                }
                pc(List.cons(head, tail), a, c, b) = (simple_graph_walk(g, a, b, List.cons(head, tail)) and
                    List.cons(head, tail).contains(c) implies
                        exists(pre: List[V], suf: List[V]) {
                            List.cons(head, tail) = pre + suf and
                                simple_graph_walk(g, a, c, pre) and simple_graph_walk(g, c, b, suf)
                        })
                pc(List.cons(head, tail), a, c, b)
            }
            p(List.cons(head, tail)) = forall(a: V, c: V, b: V) {
                pc(List.cons(head, tail), a, c, b)
            }
            p(List.cons(head, tail))
        }
    }
    forall(head: V, tail: List[V]) { p(tail) implies p(List.cons(head, tail)) }
    function[T0](x0: List[T0] -> Bool) {
        not forall(x1: T0, x2: List[T0]) { not x0(x2) or x0(List.cons(x1, x2)) } or
        not x0(List.nil[T0]) or forall(x3: List[T0]) { x0(x3) } = true
    }[V](p)
    function[T0](x0: List[T0] -> Bool, x1: List[T0]) {
        not forall(x2: T0, x3: List[T0]) { not x0(x3) or x0(List.cons(x2, x3)) } or
        not x0(List.nil[T0]) or x0(x1)
    }[V](p, steps)
    p(steps)
    pc(steps, a0, c0, b0)
    if simple_graph_walk(g, a0, b0, steps) and steps.contains(c0) {
        pc(steps, a0, c0, b0) = (simple_graph_walk(g, a0, b0, steps) and steps.contains(c0) implies
            exists(pre: List[V], suf: List[V]) {
                steps = pre + suf and simple_graph_walk(g, a0, c0, pre) and simple_graph_walk(g, c0, b0, suf)
            })
        exists(pre: List[V], suf: List[V]) {
            steps = pre + suf and simple_graph_walk(g, a0, c0, pre) and simple_graph_walk(g, c0, b0, suf)
        }
    }
}

/// The finish of a walk is determined by its target list.
theorem walk_deterministic_finish[V](g: SimpleGraph[V], a0: V, y0: V, z0: V, steps: List[V]) {
    simple_graph_walk(g, a0, y0, steps) and simple_graph_walk(g, a0, z0, steps) implies y0 = z0
} by {
    define pc(xs: List[V], a: V, y: V, z: V) -> Bool {
        simple_graph_walk(g, a, y, xs) and simple_graph_walk(g, a, z, xs) implies y = z
    }

    define p(xs: List[V]) -> Bool {
        forall(a: V, y: V, z: V) {
            pc(xs, a, y, z)
        }
    }

    forall(a: V, y: V, z: V) {
        if simple_graph_walk(g, a, y, List.nil[V]) and simple_graph_walk(g, a, z, List.nil[V]) {
            simple_graph_walk_nil(g, a, y)
            simple_graph_walk_nil(g, a, z)
            y = a
            z = a
            y = z
        }
        pc(List.nil[V], a, y, z)
    }
    p(List.nil[V])

    forall(head: V, tail: List[V]) {
        if p(tail) {
            forall(a: V, y: V, z: V) {
                if simple_graph_walk(g, a, y, List.cons(head, tail)) and
                    simple_graph_walk(g, a, z, List.cons(head, tail)) {
                    simple_graph_walk_cons_iff(g, a, y, head, tail)
                    simple_graph_walk_cons_iff(g, a, z, head, tail)
                    simple_graph_walk(g, head, y, tail)
                    simple_graph_walk(g, head, z, tail)
                    pc(tail, head, y, z)
                    pc(tail, head, y, z) = (simple_graph_walk(g, head, y, tail) and
                        simple_graph_walk(g, head, z, tail) implies y = z)
                    y = z
                }
                pc(List.cons(head, tail), a, y, z)
            }
            p(List.cons(head, tail))
        }
    }
    forall(head: V, tail: List[V]) { p(tail) implies p(List.cons(head, tail)) }
    function[T0](x0: List[T0] -> Bool) {
        not forall(x1: T0, x2: List[T0]) { not x0(x2) or x0(List.cons(x1, x2)) } or
        not x0(List.nil[T0]) or forall(x3: List[T0]) { x0(x3) } = true
    }[V](p)
    function[T0](x0: List[T0] -> Bool, x1: List[T0]) {
        not forall(x2: T0, x3: List[T0]) { not x0(x3) or x0(List.cons(x2, x3)) } or
        not x0(List.nil[T0]) or x0(x1)
    }[V](p, steps)
    p(steps)
    pc(steps, a0, y0, z0)
    if simple_graph_walk(g, a0, y0, steps) and simple_graph_walk(g, a0, z0, steps) {
        pc(steps, a0, y0, z0) = (simple_graph_walk(g, a0, y0, steps) and
            simple_graph_walk(g, a0, z0, steps) implies y0 = z0)
        y0 = z0
    }
}

/// The finish of a walk appears in its vertex list.
theorem walk_finish_in_vertex_list[V](g: SimpleGraph[V], a0: V, b0: V, steps: List[V]) {
    simple_graph_walk(g, a0, b0, steps) implies List.cons(a0, steps).contains(b0)
} by {
    define pc(xs: List[V], a: V, b: V) -> Bool {
        simple_graph_walk(g, a, b, xs) implies List.cons(a, xs).contains(b)
    }

    define p(xs: List[V]) -> Bool {
        forall(a: V, b: V) {
            pc(xs, a, b)
        }
    }

    forall(a: V, b: V) {
        if simple_graph_walk(g, a, b, List.nil[V]) {
            simple_graph_walk_nil(g, a, b)
            a = b
            List.cons(a, List.nil[V]).contains(a)
            List.cons(a, List.nil[V]).contains(b)
        }
        pc(List.nil[V], a, b)
    }
    p(List.nil[V])

    forall(head: V, tail: List[V]) {
        if p(tail) {
            forall(a: V, b: V) {
                if simple_graph_walk(g, a, b, List.cons(head, tail)) {
                    simple_graph_walk_cons_iff(g, a, b, head, tail)
                    simple_graph_walk(g, head, b, tail)
                    pc(tail, head, b)
                    pc(tail, head, b) = (simple_graph_walk(g, head, b, tail) implies
                        List.cons(head, tail).contains(b))
                    List.cons(head, tail).contains(b)
                    List.cons(a, List.cons(head, tail)).contains(b)
                }
                pc(List.cons(head, tail), a, b)
            }
            p(List.cons(head, tail))
        }
    }
    forall(head: V, tail: List[V]) { p(tail) implies p(List.cons(head, tail)) }
    function[T0](x0: List[T0] -> Bool) {
        not forall(x1: T0, x2: List[T0]) { not x0(x2) or x0(List.cons(x1, x2)) } or
        not x0(List.nil[T0]) or forall(x3: List[T0]) { x0(x3) } = true
    }[V](p)
    function[T0](x0: List[T0] -> Bool, x1: List[T0]) {
        not forall(x2: T0, x3: List[T0]) { not x0(x3) or x0(List.cons(x2, x3)) } or
        not x0(List.nil[T0]) or x0(x1)
    }[V](p, steps)
    p(steps)
    pc(steps, a0, b0)
    if simple_graph_walk(g, a0, b0, steps) {
        pc(steps, a0, b0) = (simple_graph_walk(g, a0, b0, steps) implies List.cons(a0, steps).contains(b0))
        List.cons(a0, steps).contains(b0)
    }
}

/// The suffix of a unique list is unique.
theorem unique_suffix_of_add[T](prefix0: List[T], suffix0: List[T]) {
    ((prefix0 + suffix0).is_unique implies suffix0.is_unique)
} by {
    define pc(xs: List[T]) -> Bool {
        forall(suffix: List[T]) {
            (xs + suffix).is_unique implies suffix.is_unique
        }
    }
    forall(suffix: List[T]) {
        ((List.nil[T] + suffix).is_unique implies suffix.is_unique)
    }
    pc(List.nil[T])
    forall(head: T, tail: List[T]) {
        if pc(tail) {
            pc(tail) = forall(suffix: List[T]) {
                ((tail + suffix).is_unique implies suffix.is_unique)
            }
            forall(suffix: List[T]) {
                if (List.cons(head, tail) + suffix).is_unique {
                    List.cons(head, tail) + suffix = List.cons(head, tail + suffix)
                    List.cons(head, tail + suffix).is_unique
                    unique_cons_parts(head, tail + suffix)
                    (tail + suffix).is_unique
                    (tail + suffix).is_unique implies suffix.is_unique
                    suffix.is_unique
                }
                ((List.cons(head, tail) + suffix).is_unique implies suffix.is_unique)
            }
            pc(List.cons(head, tail))
        }
        pc(tail) implies pc(List.cons(head, tail))
    }
    function[T0](x0: List[T0] -> Bool) {
        not forall(x1: T0, x2: List[T0]) { not x0(x2) or x0(List.cons(x1, x2)) } or
        not x0(List.nil[T0]) or forall(x3: List[T0]) { x0(x3) } = true
    }[T](pc)
    function[T0](x0: List[T0] -> Bool, x1: List[T0]) {
        not forall(x2: T0, x3: List[T0]) { not x0(x3) or x0(List.cons(x2, x3)) } or
        not x0(List.nil[T0]) or x0(x1)
    }[T](pc, prefix0)
    pc(prefix0)
    pc(prefix0) = forall(suffix: List[T]) {
        ((prefix0 + suffix).is_unique implies suffix.is_unique)
    }
    ((prefix0 + suffix0).is_unique implies suffix0.is_unique)
}

/// Appending a fresh element keeps a unique list unique.
theorem unique_add_singleton[T](list0: List[T], item0: T) {
    (list0.is_unique and not list0.contains(item0) implies
        (list0 + List.singleton(item0)).is_unique)
} by {
    define pc(xs: List[T]) -> Bool {
        forall(item: T) {
            xs.is_unique and not xs.contains(item) implies
                (xs + List.singleton(item)).is_unique
        }
    }
    forall(item: T) {
        (List.nil[T].is_unique and not List.nil[T].contains(item) implies
            (List.nil[T] + List.singleton(item)).is_unique)
    }
    pc(List.nil[T])
    forall(head: T, tail: List[T]) {
        if pc(tail) {
            pc(tail) = forall(item: T) {
                tail.is_unique and not tail.contains(item) implies
                    (tail + List.singleton(item)).is_unique
            }
            forall(item: T) {
                if List.cons(head, tail).is_unique and not List.cons(head, tail).contains(item) {
                    unique_cons_parts(head, tail)
                    tail.is_unique
                    not tail.contains(head)
                    if tail.contains(item) {
                        List.cons(head, tail).contains(item) = (head = item or tail.contains(item))
                        List.cons(head, tail).contains(item)
                        false
                    }
                    not tail.contains(item)
                    (tail + List.singleton(item)).is_unique
                    List.cons(head, tail) + List.singleton(item) = List.cons(head, tail + List.singleton(item))
                    if head = item {
                        List.cons(head, tail).contains(item) = (head = item or tail.contains(item))
                        List.cons(head, tail).contains(item)
                        false
                    }
                    head != item
                    if List.singleton(item).contains(head) {
                        singleton_contains_imp_eq(item, head)
                        List.singleton(item).contains(head) implies head = item
                        head = item
                        false
                    }
                    not List.singleton(item).contains(head)
                    add_contains_or(tail, List.singleton(item), head)
                    ((tail + List.singleton(item)).contains(head) implies
                        tail.contains(head) or List.singleton(item).contains(head))
                    not (tail + List.singleton(item)).contains(head)
                    unique_cons_intro(head, tail + List.singleton(item))
                    List.cons(head, tail + List.singleton(item)).is_unique
                    (List.cons(head, tail) + List.singleton(item)).is_unique
                }
                (List.cons(head, tail).is_unique and not List.cons(head, tail).contains(item) implies
                    (List.cons(head, tail) + List.singleton(item)).is_unique)
            }
            pc(List.cons(head, tail))
        }
        pc(tail) implies pc(List.cons(head, tail))
    }
    function[T0](x0: List[T0] -> Bool) {
        not forall(x1: T0, x2: List[T0]) { not x0(x2) or x0(List.cons(x1, x2)) } or
        not x0(List.nil[T0]) or forall(x3: List[T0]) { x0(x3) } = true
    }[T](pc)
    function[T0](x0: List[T0] -> Bool, x1: List[T0]) {
        not forall(x2: T0, x3: List[T0]) { not x0(x3) or x0(List.cons(x2, x3)) } or
        not x0(List.nil[T0]) or x0(x1)
    }[T](pc, list0)
    pc(list0)
    pc(list0) = forall(item: T) {
        list0.is_unique and not list0.contains(item) implies
            (list0 + List.singleton(item)).is_unique
    }
    (list0.is_unique and not list0.contains(item0) implies
        (list0 + List.singleton(item0)).is_unique)
}

/// An element of both parts of a concatenation makes it non-unique.
theorem dup_in_add_not_unique[T](left0: List[T], right0: List[T], item0: T) {
    (left0.contains(item0) and right0.contains(item0) implies not (left0 + right0).is_unique)
} by {
    define pc(xs: List[T]) -> Bool {
        forall(right: List[T], item: T) {
            xs.contains(item) and right.contains(item) implies not (xs + right).is_unique
        }
    }
    forall(right: List[T], item: T) {
        (List.nil[T].contains(item) and right.contains(item) implies
            not (List.nil[T] + right).is_unique) = true
        eq_true_elim(List.nil[T].contains(item) and right.contains(item) implies
            not (List.nil[T] + right).is_unique)
        (List.nil[T].contains(item) and right.contains(item) implies
            not (List.nil[T] + right).is_unique)
    }
    pc(List.nil[T])
    forall(head: T, tail: List[T]) {
        if pc(tail) {
            pc(tail) = forall(right: List[T], item: T) {
                tail.contains(item) and right.contains(item) implies not (tail + right).is_unique
            }
            forall(right: List[T], item: T) {
                if List.cons(head, tail).contains(item) and right.contains(item) {
                    if head = item {
                        List.cons(head, tail) + right = List.cons(head, tail + right)
                        right.contains(item)
                        add_contains_right(tail, right, item)
                        (tail + right).contains(item)
                        head = item
                        (tail + right).contains(head)
                        if (List.cons(head, tail) + right).is_unique {
                            unique_cons_head_not_in_tail(head, tail + right)
                            not (tail + right).contains(head)
                            false
                        }
                        not_intro((List.cons(head, tail) + right).is_unique)
                        not (List.cons(head, tail) + right).is_unique
                    }
                    if head != item {
                        List.cons(head, tail).contains(item) = (head = item or tail.contains(item))
                        tail.contains(item)
                        pc(tail)
                        pc(tail) = forall(r: List[T], it: T) {
                            (tail.contains(it) and r.contains(it) implies not (tail + r).is_unique)
                        }
                        (tail.contains(item) and right.contains(item) implies
                            not (tail + right).is_unique)
                        not (tail + right).is_unique
                        if (List.cons(head, tail) + right).is_unique {
                            List.cons(head, tail) + right = List.cons(head, tail + right)
                            unique_cons_parts(head, tail + right)
                            (tail + right).is_unique
                            false
                        }
                        not_intro((List.cons(head, tail) + right).is_unique)
                        not (List.cons(head, tail) + right).is_unique
                    }
                    not (List.cons(head, tail) + right).is_unique
                }
                (List.cons(head, tail).contains(item) and right.contains(item) implies
                    not (List.cons(head, tail) + right).is_unique)
            }
            pc(List.cons(head, tail))
        }
        pc(tail) implies pc(List.cons(head, tail))
    }
    function[T0](x0: List[T0] -> Bool) {
        not forall(x1: T0, x2: List[T0]) { not x0(x2) or x0(List.cons(x1, x2)) } or
        not x0(List.nil[T0]) or forall(x3: List[T0]) { x0(x3) } = true
    }[T](pc)
    function[T0](x0: List[T0] -> Bool, x1: List[T0]) {
        not forall(x2: T0, x3: List[T0]) { not x0(x3) or x0(List.cons(x2, x3)) } or
        not x0(List.nil[T0]) or x0(x1)
    }[T](pc, left0)
    pc(left0)
    pc(left0) = forall(right: List[T], item: T) {
        (left0.contains(item) and right.contains(item) implies not (left0 + right).is_unique)
    }
    (left0.contains(item0) and right0.contains(item0) implies not (left0 + right0).is_unique)
}

/// A target of a cons that is not the head is a target of the tail.
theorem cons_contains_imp_tail_contains[T](head: T, tail: List[T], item: T) {
    List.cons(head, tail).contains(item) and item != head implies tail.contains(item)
} by {
    if List.cons(head, tail).contains(item) and item != head {
        List.cons(head, tail).contains(item) = (head = item or tail.contains(item))
        if head = item {
            item != head
            false
        }
        tail.contains(item)
    }
}

/// A nonempty walk has a last edge: some vertex of `s` is adjacent to the finish.
///
/// Read off the last edge of the walk; both endpoints lie in `s` by hypothesis.
theorem walk_last_neighbor_in_s[V](g: SimpleGraph[V], s: FiniteSet[V], a0: V, b0: V, steps: List[V]) {
    simple_graph_walk(g, a0, b0, steps) and
    s.contains(a0) and
    (forall(x: V) { steps.contains(x) implies s.contains(x) }) and
    exists(w: V, rest: List[V]) { steps = List.cons(w, rest) }
    implies exists(u: V) { s.contains(u) and g.adj(u, b0) }
} by {
    define pc(xs: List[V], a: V, b: V) -> Bool {
        simple_graph_walk(g, a, b, xs) and
        s.contains(a) and
        (forall(x: V) { xs.contains(x) implies s.contains(x) }) and
        exists(w: V, rest: List[V]) { xs = List.cons(w, rest) }
        implies exists(u: V) { s.contains(u) and g.adj(u, b) }
    }

    define p(xs: List[V]) -> Bool {
        forall(a: V, b: V) {
            pc(xs, a, b)
        }
    }

    forall(a: V, b: V) {
        (simple_graph_walk(g, a, b, List.nil[V]) and
            s.contains(a) and
            (forall(x: V) { List.nil[V].contains(x) implies s.contains(x) }) and
            exists(w: V, rest: List[V]) { List.nil[V] = List.cons(w, rest) }
            implies exists(u: V) { s.contains(u) and g.adj(u, b) }) = true
        eq_true_elim(simple_graph_walk(g, a, b, List.nil[V]) and
            s.contains(a) and
            (forall(x: V) { List.nil[V].contains(x) implies s.contains(x) }) and
            exists(w: V, rest: List[V]) { List.nil[V] = List.cons(w, rest) }
            implies exists(u: V) { s.contains(u) and g.adj(u, b) })
        (simple_graph_walk(g, a, b, List.nil[V]) and
            s.contains(a) and
            (forall(x: V) { List.nil[V].contains(x) implies s.contains(x) }) and
            exists(w: V, rest: List[V]) { List.nil[V] = List.cons(w, rest) }
            implies exists(u: V) { s.contains(u) and g.adj(u, b) })
        pc(List.nil[V], a, b)
    }
    p(List.nil[V])

    forall(head: V, tail: List[V]) {
        if p(tail) {
            forall(a: V, b: V) {
                if simple_graph_walk(g, a, b, List.cons(head, tail)) and
                    s.contains(a) and
                    (forall(x: V) { List.cons(head, tail).contains(x) implies s.contains(x) }) and
                    exists(w: V, rest: List[V]) { List.cons(head, tail) = List.cons(w, rest) } {
                    simple_graph_walk_cons_iff(g, a, b, head, tail)
                    simple_graph_walk(g, a, b, List.cons(head, tail)) =
                        (g.adj(a, head) and simple_graph_walk(g, head, b, tail))
                    simple_graph_walk(g, head, b, tail)
                    List.cons(head, tail).contains(head)
                    List.cons(head, tail).contains(head) implies s.contains(head)
                    s.contains(head)
                    if tail = List.nil[V] {
                        simple_graph_walk_nil(g, head, b)
                        simple_graph_walk(g, head, b, List.nil[V]) = (head = b)
                        head = b
                        g.adj(a, head)
                        s.contains(a)
                        exists(u: V) { s.contains(u) and g.adj(u, b) }
                    }
                    if tail != List.nil[V] {
                        forall(x: V) {
                            if tail.contains(x) {
                                List.cons(head, tail).contains(x) = (head = x or tail.contains(x))
                                List.cons(head, tail).contains(x) implies s.contains(x)
                                s.contains(x)
                            }
                            tail.contains(x) implies s.contains(x)
                        }
                        pc(tail, head, b)
                        pc(tail, head, b) = (simple_graph_walk(g, head, b, tail) and
                            s.contains(head) and
                            (forall(x: V) { tail.contains(x) implies s.contains(x) }) and
                            exists(w: V, rest: List[V]) { tail = List.cons(w, rest) }
                            implies exists(u: V) { s.contains(u) and g.adj(u, b) })
                        match tail {
                            List.nil {
                                List.nil[V] != List.nil[V]
                                false
                            }
                            List.cons(w, rest) {
                                exists(x: V, r: List[V]) { tail = List.cons(x, r) }
                            }
                        }
                        exists(u: V) { s.contains(u) and g.adj(u, b) }
                    }
                    if tail = List.nil[V] {
                        exists(u: V) { s.contains(u) and g.adj(u, b) }
                    }
                    if tail != List.nil[V] {
                        exists(u: V) { s.contains(u) and g.adj(u, b) }
                    }
                    exists(u: V) { s.contains(u) and g.adj(u, b) }
                }
                pc(List.cons(head, tail), a, b)
            }
            p(List.cons(head, tail))
        }
    }
    forall(head: V, tail: List[V]) { p(tail) implies p(List.cons(head, tail)) }
    function[T0](x0: List[T0] -> Bool) {
        not forall(x1: T0, x2: List[T0]) { not x0(x2) or x0(List.cons(x1, x2)) } or
        not x0(List.nil[T0]) or forall(x3: List[T0]) { x0(x3) } = true
    }[V](p)
    function[T0](x0: List[T0] -> Bool, x1: List[T0]) {
        not forall(x2: T0, x3: List[T0]) { not x0(x3) or x0(List.cons(x2, x3)) } or
        not x0(List.nil[T0]) or x0(x1)
    }[V](p, steps)
    p(steps)
    pc(steps, a0, b0)
    if simple_graph_walk(g, a0, b0, steps) and
        s.contains(a0) and
        (forall(x: V) { steps.contains(x) implies s.contains(x) }) and
        exists(w: V, rest: List[V]) { steps = List.cons(w, rest) } {
        pc(steps, a0, b0) = (simple_graph_walk(g, a0, b0, steps) and
            s.contains(a0) and
            (forall(x: V) { steps.contains(x) implies s.contains(x) }) and
            exists(w: V, rest: List[V]) { steps = List.cons(w, rest) }
            implies exists(u: V) { s.contains(u) and g.adj(u, b0) })
        exists(u: V) { s.contains(u) and g.adj(u, b0) }
    }
}

/// A vertex that is not a leaf but has a neighbor has two distinct neighbors.
theorem not_is_leaf_two_neighbors[V](g: SimpleGraph[V], s: FiniteSet[V], v: V, u0: V) {
    not is_leaf(g, s, v) and s.contains(u0) and g.adj(v, u0) implies
        exists(c: V, d: V) {
            s.contains(c) and s.contains(d) and c != d and g.adj(v, c) and g.adj(v, d)
        }
} by {
    if not is_leaf(g, s, v) and s.contains(u0) and g.adj(v, u0) {
        is_leaf(g, s, v) = exists(u: V) {
            s.contains(u) and g.adj(v, u) and
                forall(w: V) {
                    s.contains(w) and g.adj(v, w) implies w = u
                }
        }
        if exists(u: V) {
            s.contains(u) and g.adj(v, u) and
                forall(w: V) {
                    s.contains(w) and g.adj(v, w) implies w = u
                }
        } {
            false
        }
        not (exists(u: V) {
            s.contains(u) and g.adj(v, u) and
                forall(w: V) {
                    s.contains(w) and g.adj(v, w) implies w = u
                }
        })
        not_exists_forward(function(u: V) {
            s.contains(u) and g.adj(v, u) and
                forall(w: V) {
                    s.contains(w) and g.adj(v, w) implies w = u
                }
        })
        forall(u: V) {
            not (s.contains(u) and g.adj(v, u) and
                forall(w: V) {
                    s.contains(w) and g.adj(v, w) implies w = u
                })
        }
        not (s.contains(u0) and g.adj(v, u0) and
            forall(w: V) {
                s.contains(w) and g.adj(v, w) implies w = u0
            })
        if forall(w: V) {
            s.contains(w) and g.adj(v, w) implies w = u0
        } {
            s.contains(u0) and g.adj(v, u0) and
                forall(w: V) {
                    s.contains(w) and g.adj(v, w) implies w = u0
                }
            false
        }
        not (forall(w: V) {
            s.contains(w) and g.adj(v, w) implies w = u0
        })
        not_forall_imp_exists_not(function(w: V) {
            s.contains(w) and g.adj(v, w) implies w = u0
        })
        exists(w: V) {
            not (s.contains(w) and g.adj(v, w) implies w = u0)
        }
        let (c: V) satisfy {
            not (s.contains(c) and g.adj(v, c) implies c = u0)
        }
        not_implies(s.contains(c) and g.adj(v, c), c = u0)
        not (s.contains(c) and g.adj(v, c) implies c = u0) =
            (s.contains(c) and g.adj(v, c) and not (c = u0))
        s.contains(c) and g.adj(v, c) and c != u0
        s.contains(c)
        g.adj(v, c)
        c != u0
        exists(x: V, y: V) {
            s.contains(x) and s.contains(y) and x != y and g.adj(v, x) and g.adj(v, y)
        }
    }
}

/// A cycle of `g` lying entirely in `s` contradicts acyclicity on `s`.
theorem is_acyclic_cycle_contradiction[V](g: SimpleGraph[V], s: FiniteSet[V], start: V, steps: List[V]) {
    not (is_acyclic(g, s) and s.contains(start) and simple_graph_cycle(g, start, steps) and
        (forall(x: V) { steps.contains(x) implies s.contains(x) }))
} by {
    if is_acyclic(g, s) and s.contains(start) and simple_graph_cycle(g, start, steps) and
        (forall(x: V) { steps.contains(x) implies s.contains(x) }) {
        is_acyclic(g, s) = forall(a: V, b: List[V]) {
            not (s.contains(a) and simple_graph_cycle(g, a, b) and
                (forall(x: V) { b.contains(x) implies s.contains(x) }))
        }
        s.contains(start) and simple_graph_cycle(g, start, steps) and
            (forall(x: V) { steps.contains(x) implies s.contains(x) })
        false
    }
    not (is_acyclic(g, s) and s.contains(start) and simple_graph_cycle(g, start, steps) and
        (forall(x: V) { steps.contains(x) implies s.contains(x) }))
}

/// Dropping the last element of a singleton-suffixed list recovers the prefix.
theorem add_singleton_drop_last[T](prefix0: List[T], item0: T) {
    (prefix0 + List.singleton(item0)).drop_last(Nat.1) = prefix0
} by {
    define pc(xs: List[T]) -> Bool {
        forall(item: T) {
            (xs + List.singleton(item)).drop_last(Nat.1) = xs
        }
    }
    forall(item: T) {
        List.nil[T] + List.singleton(item) = List.singleton(item)
        List.singleton(item) = List.cons(item, List.nil[T])
        List.nil[T].length = Nat.0
        Nat.0 < Nat.1
        List.cons(item, List.nil[T]).drop_last(Nat.1) = List.nil[T]
        (List.nil[T] + List.singleton(item)).drop_last(Nat.1) = List.nil[T]
    }
    pc(List.nil[T])
    forall(head: T, tail: List[T]) {
        if pc(tail) {
            pc(tail) = forall(item: T) {
                (tail + List.singleton(item)).drop_last(Nat.1) = tail
            }
            forall(item: T) {
                List.cons(head, tail) + List.singleton(item) = List.cons(head, tail + List.singleton(item))
                add_length(tail, List.singleton(item))
                (tail + List.singleton(item)).length = tail.length + List.singleton(item).length
                List.cons(item, List.nil[T]).length = List.nil[T].length + Nat.1
                List.nil[T].length = Nat.0
                List.cons(item, List.nil[T]).length = Nat.1
                List.singleton(item) = List.cons(item, List.nil[T])
                List.singleton(item).length = Nat.1
                (tail + List.singleton(item)).length = tail.length + Nat.1
                Nat.0 <= tail.length
                lte_suc_suc(Nat.0, tail.length)
                Nat.1 <= tail.length.suc
                tail.length + Nat.1 = tail.length.suc
                Nat.1 <= tail.length + Nat.1
                lte_imp_not_lt(Nat.1, tail.length + Nat.1)
                not (tail.length + Nat.1 < Nat.1)
                not ((tail + List.singleton(item)).length < Nat.1)
                List.cons(head, tail + List.singleton(item)).drop_last(Nat.1) =
                    List.cons(head, (tail + List.singleton(item)).drop_last(Nat.1))
                (tail + List.singleton(item)).drop_last(Nat.1) = tail
                List.cons(head, tail + List.singleton(item)).drop_last(Nat.1) = List.cons(head, tail)
                (List.cons(head, tail) + List.singleton(item)).drop_last(Nat.1) =
                    List.cons(head, tail)
            }
            pc(List.cons(head, tail))
        }
        pc(tail) implies pc(List.cons(head, tail))
    }
    function[T0](x0: List[T0] -> Bool) {
        not forall(x1: T0, x2: List[T0]) { not x0(x2) or x0(List.cons(x1, x2)) } or
        not x0(List.nil[T0]) or forall(x3: List[T0]) { x0(x3) } = true
    }[T](pc)
    function[T0](x0: List[T0] -> Bool, x1: List[T0]) {
        not forall(x2: T0, x3: List[T0]) { not x0(x3) or x0(List.cons(x2, x3)) } or
        not x0(List.nil[T0]) or x0(x1)
    }[T](pc, prefix0)
    pc(prefix0)
    pc(prefix0) = forall(item: T) {
        (prefix0 + List.singleton(item)).drop_last(Nat.1) = prefix0
    }
    (prefix0 + List.singleton(item0)).drop_last(Nat.1) = prefix0
}

// NOTE (leaf theorem, stretch): the cycle-contradiction lemma below is the last
// missing piece of the leaf-existence proof (every finite tree with at least two
// vertices has a leaf). Its statement is verified by inspection; the proof body
// could not be closed: the prover refuses ex falso from `not P` together with `P`
// derived from a cited theorem (it reports "inconsistent assumptions"). The rest of
// the machinery (path lengths, walk splitting, determinism, cycle construction)
// is fully verified above. Commented out per AGENTS.md after many attempts.
// /// A split suffix of length at least two, closed by the edge to its start, is a cycle.
// ///
// /// The walk from `c` to `finish` followed by the edge `finish - c` is a closed walk of
// /// length at least three. The path's uniqueness makes its targets distinct and `c` fresh,
// /// so it is a genuine cycle lying in `s`, contradicting acyclicity.
// theorem split_suffix_cycle_contradiction[V](g: SimpleGraph[V], s: FiniteSet[V], start: V, c: V, finish: V, steps: List[V], p: List[V], suf: List[V]) {
//     is_tree(g, s) and
//     simple_graph_path(g, start, finish, steps) and
//     s.contains(c) and
//     (forall(x: V) { steps.contains(x) implies s.contains(x) }) and
//     steps = p + suf and
//     simple_graph_walk(g, start, c, p) and
//     simple_graph_walk(g, c, finish, suf) and
//     g.adj(finish, c) and
//     Nat.2 <= suf.length
//     implies false
// } by {
//     if is_tree(g, s) and
//         simple_graph_path(g, start, finish, steps) and
//         s.contains(c) and
//         (forall(x: V) { steps.contains(x) implies s.contains(x) }) and
//         steps = p + suf and
//         simple_graph_walk(g, start, c, p) and
//         simple_graph_walk(g, c, finish, suf) and
//         g.adj(finish, c) and
//         Nat.2 <= suf.length {
//         tree_acyclic(g, s)
//         is_acyclic(g, s)
//         simple_graph_walk_append_adj(g, c, finish, c, suf)
//         simple_graph_walk(g, c, c, suf.append(c))
//         suf.append(c) = suf + List.singleton(c)
//         simple_graph_walk(g, c, c, suf + List.singleton(c))
//         add_length(suf, List.singleton(c))
//         (suf + List.singleton(c)).length = suf.length + List.singleton(c).length
//         List.cons(c, List.nil[V]).length = List.nil[V].length + Nat.1
//         List.nil[V].length = Nat.0
//         List.cons(c, List.nil[V]).length = Nat.1
//         List.singleton(c) = List.cons(c, List.nil[V])
//         List.singleton(c).length = Nat.1
//         (suf + List.singleton(c)).length = suf.length + Nat.1
//         lte_add_left(Nat.1, Nat.2, suf.length)
//         Nat.1 + Nat.2 <= Nat.1 + suf.length
//         lte_trans(Nat.3, Nat.1 + suf.length, (suf + List.singleton(c)).length)
//         Nat.3 <= (suf + List.singleton(c)).length
//         simple_graph_path_vertices_unique(g, start, finish, steps)
//         simple_graph_walk_vertices(start, steps).is_unique
//         simple_graph_walk_vertices(start, steps) = List.cons(start, steps)
//         List.cons(start, steps).is_unique
//         unique_suffix_of_add(p, suf)
//         steps = p + suf
//         suf.is_unique
//         if suf.contains(c) {
//             walk_finish_in_vertex_list(g, start, c, p)
//             List.cons(start, p).contains(c)
//             dup_in_add_not_unique(List.cons(start, p), suf, c)
//             not (List.cons(start, p) + suf).is_unique
//             List.cons(start, steps) = List.cons(start, p + suf)
//             List.cons(start, p + suf) = List.cons(start, p) + suf
//             List.cons(start, steps) = List.cons(start, p) + suf
//             List.cons(start, steps).is_unique
//             false
//         }
//         not suf.contains(c)
//         unique_add_singleton(suf, c)
//         (suf + List.singleton(c)).is_unique
//         simple_graph_closed_walk(g, c, suf + List.singleton(c)) =
//             (simple_graph_walk(g, c, c, suf + List.singleton(c)) and
//                 (suf + List.singleton(c)).length > Nat.0)
//         lt_and_lte(Nat.0, Nat.3, (suf + List.singleton(c)).length)
//         Nat.0 < Nat.3
//         (suf + List.singleton(c)).length > Nat.0
//         simple_graph_closed_walk(g, c, suf + List.singleton(c))
//         simple_graph_cycle(g, c, suf + List.singleton(c)) =
//             (simple_graph_closed_walk(g, c, suf + List.singleton(c)) and
//                 Nat.3 <= (suf + List.singleton(c)).length and (suf + List.singleton(c)).is_unique)
//         simple_graph_cycle(g, c, suf + List.singleton(c))
//         forall(x: V) {
//             if (suf + List.singleton(c)).contains(x) {
//                 add_contains_or(suf, List.singleton(c), x)
//                 (suf + List.singleton(c)).contains(x) implies suf.contains(x) or List.singleton(c).contains(x)
//                 if suf.contains(x) or List.singleton(c).contains(x) {
//                     if suf.contains(x) {
//                         steps = p + suf
//                         add_contains_left(p, suf, x)
//                         (p + suf).contains(x)
//                         steps.contains(x)
//                         steps.contains(x) implies s.contains(x)
//                         s.contains(x)
//                     }
//                     if not suf.contains(x) {
//                         List.singleton(c).contains(x)
//                         singleton_contains_imp_eq(c, x)
//                         List.singleton(c).contains(x) implies x = c
//                         x = c
//                         s.contains(c)
//                         x = c
//                         s.contains(x)
//                     }
//                     s.contains(x)
//                 }
//                 s.contains(x)
//             }
//             (suf + List.singleton(c)).contains(x) implies s.contains(x)
//         }
//         if is_acyclic(g, s) and s.contains(c) and
//             simple_graph_cycle(g, c, suf + List.singleton(c)) and
//             (forall(x: V) { (suf + List.singleton(c)).contains(x) implies s.contains(x) }) {
//             is_acyclic_cycle_contradiction(g, s, c, suf + List.singleton(c))
//             not (is_acyclic(g, s) and s.contains(c) and
//                 simple_graph_cycle(g, c, suf + List.singleton(c)) and
//                 (forall(x: V) { (suf + List.singleton(c)).contains(x) implies s.contains(x) }))
//             is_acyclic(g, s) and s.contains(c) and
//                 simple_graph_cycle(g, c, suf + List.singleton(c)) and
//                 (forall(x: V) { (suf + List.singleton(c)).contains(x) implies s.contains(x) })
//             false
//         }
//     }
// }
