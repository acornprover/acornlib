from nat import Nat
from data.basic.functions import is_injective_fn, compose
from graph.simple_graph import SimpleGraph, is_graph_hom, is_graph_embedding, reflects_graph_adj,
    reflects_graph_adj_apply, graph_embedding_is_hom, graph_embedding_reflects_adj,
    graph_embedding_is_injective, is_graph_iso_pair, is_graph_iso_pair_reflects_adj,
    is_graph_iso_pair_map_is_injective, are_inverse_vertex_maps

numerals Nat

/// An embedding preserves adjacency in the forward direction.
///
/// Restates `is_graph_hom` at a pair of vertices, which is the form every argument below
/// actually uses.
theorem graph_embedding_adj_forward[V, W](
    g: SimpleGraph[V], h: SimpleGraph[W], f: V -> W, x: V, y: V
) {
    is_graph_embedding(g, h, f) and g.adj(x, y) implies h.adj(f(x), f(y))
} by {
    if is_graph_embedding(g, h, f) and g.adj(x, y) {
        graph_embedding_is_hom(g, h, f)
        is_graph_hom(g, h, f)
        is_graph_hom(g, h, f) = forall(a: V, b: V) {
            g.adj(a, b) implies h.adj(f(a), f(b))
        }
        forall(a: V, b: V) {
            g.adj(a, b) implies h.adj(f(a), f(b))
        }
        (g.adj(x, y) implies h.adj(f(x), f(y)))
        h.adj(f(x), f(y))
    }
}

/// An embedding reflects adjacency at a pair of vertices.
theorem graph_embedding_adj_backward[V, W](
    g: SimpleGraph[V], h: SimpleGraph[W], f: V -> W, x: V, y: V
) {
    is_graph_embedding(g, h, f) and h.adj(f(x), f(y)) implies g.adj(x, y)
} by {
    if is_graph_embedding(g, h, f) and h.adj(f(x), f(y)) {
        graph_embedding_reflects_adj(g, h, f)
        reflects_graph_adj(g, h, f)
        reflects_graph_adj_apply(g, h, f, x, y)
        g.adj(x, y)
    }
}

/// The image of an embedding sees exactly the source adjacency.
///
/// This is what makes the copy *induced* rather than merely a subgraph: non-edges are
/// carried to non-edges as well.
theorem graph_embedding_adj_eq[V, W](
    g: SimpleGraph[V], h: SimpleGraph[W], f: V -> W, x: V, y: V
) {
    is_graph_embedding(g, h, f) implies h.adj(f(x), f(y)) = g.adj(x, y)
} by {
    if is_graph_embedding(g, h, f) {
        graph_embedding_adj_forward(g, h, f, x, y)
        graph_embedding_adj_backward(g, h, f, x, y)
        (g.adj(x, y) implies h.adj(f(x), f(y)))
        (h.adj(f(x), f(y)) implies g.adj(x, y))
        h.adj(f(x), f(y)) = g.adj(x, y)
    }
}

/// The three conditions give an embedding.
theorem is_graph_embedding_intro[V, W](
    g: SimpleGraph[V], h: SimpleGraph[W], f: V -> W
) {
    is_graph_hom(g, h, f) and reflects_graph_adj(g, h, f) and is_injective_fn(f)
        implies is_graph_embedding(g, h, f)
} by {
    if is_graph_hom(g, h, f) and reflects_graph_adj(g, h, f) and is_injective_fn(f) {
        is_graph_embedding(g, h, f) =
            (is_graph_hom(g, h, f) and reflects_graph_adj(g, h, f) and is_injective_fn(f))
        is_graph_embedding(g, h, f)
    }
}

/// An adjacency equality plus injectivity is an embedding.
///
/// The convenient entry point: rather than assembling the three parts separately, supply the
/// single biconditional that says the map is induced.
theorem is_graph_embedding_of_adj_eq[V, W](
    g: SimpleGraph[V], h: SimpleGraph[W], f: V -> W
) {
    (forall(x: V, y: V) { h.adj(f(x), f(y)) = g.adj(x, y) }) and is_injective_fn(f)
        implies is_graph_embedding(g, h, f)
} by {
    if (forall(x: V, y: V) { h.adj(f(x), f(y)) = g.adj(x, y) }) and is_injective_fn(f) {
        forall(a: V, b: V) {
            h.adj(f(a), f(b)) = g.adj(a, b)
            (g.adj(a, b) implies h.adj(f(a), f(b)))
        }
        is_graph_hom(g, h, f) = forall(a: V, b: V) {
            g.adj(a, b) implies h.adj(f(a), f(b))
        }
        is_graph_hom(g, h, f)
        forall(a: V, b: V) {
            h.adj(f(a), f(b)) = g.adj(a, b)
            (h.adj(f(a), f(b)) implies g.adj(a, b))
        }
        reflects_graph_adj(g, h, f) = forall(a: V, b: V) {
            h.adj(f(a), f(b)) implies g.adj(a, b)
        }
        reflects_graph_adj(g, h, f)
        is_graph_embedding_intro(g, h, f)
        is_graph_embedding(g, h, f)
    }
}

/// A composite of embeddings is an embedding.
///
/// Needed so that freeness transports along a chain: a copy inside a copy is still a copy.
theorem is_graph_embedding_compose[U, V, W](
    g: SimpleGraph[U], h: SimpleGraph[V], k: SimpleGraph[W], e: U -> V, f: V -> W
) {
    is_graph_embedding(g, h, e) and is_graph_embedding(h, k, f)
        implies is_graph_embedding(g, k, compose(f, e))
} by {
    if is_graph_embedding(g, h, e) and is_graph_embedding(h, k, f) {
        forall(x: U, y: U) {
            compose(f, e)(x) = f(e(x))
            compose(f, e)(y) = f(e(y))
            graph_embedding_adj_eq(h, k, f, e(x), e(y))
            k.adj(f(e(x)), f(e(y))) = h.adj(e(x), e(y))
            graph_embedding_adj_eq(g, h, e, x, y)
            h.adj(e(x), e(y)) = g.adj(x, y)
            k.adj(compose(f, e)(x), compose(f, e)(y)) = g.adj(x, y)
        }
        graph_embedding_is_injective(g, h, e)
        is_injective_fn(e)
        graph_embedding_is_injective(h, k, f)
        is_injective_fn(f)
        forall(x: U, y: U) {
            if compose(f, e)(x) = compose(f, e)(y) {
                f(e(x)) = f(e(y))
                is_injective_fn(f) = forall(a: V, b: V) { f(a) = f(b) implies a = b }
                forall(a: V, b: V) { f(a) = f(b) implies a = b }
                (f(e(x)) = f(e(y)) implies e(x) = e(y))
                e(x) = e(y)
                is_injective_fn(e) = forall(a: U, b: U) { e(a) = e(b) implies a = b }
                forall(a: U, b: U) { e(a) = e(b) implies a = b }
                (e(x) = e(y) implies x = y)
                x = y
            }
            (compose(f, e)(x) = compose(f, e)(y) implies x = y)
        }
        is_injective_fn(compose(f, e)) = forall(a: U, b: U) {
            compose(f, e)(a) = compose(f, e)(b) implies a = b
        }
        is_injective_fn(compose(f, e))
        is_graph_embedding_of_adj_eq(g, k, compose(f, e))
        is_graph_embedding(g, k, compose(f, e))
    }
}

/// True if `h` has an induced copy of `g`.
///
/// Induced is the strong reading: the copy must reproduce the non-edges of `g` as well as its
/// edges. Stated once here so that every forbidden-subgraph condition below is its negation.
define contains_induced[V, W](h: SimpleGraph[W], g: SimpleGraph[V]) -> Bool {
    exists(f: V -> W) {
        is_graph_embedding(g, h, f)
    }
}

/// An embedding witnesses an induced copy.
theorem contains_induced_intro[V, W](
    h: SimpleGraph[W], g: SimpleGraph[V], m: V -> W
) {
    is_graph_embedding(g, h, m) implies contains_induced(h, g)
} by {
    if is_graph_embedding(g, h, m) {
        exists(f: V -> W) {
            f = m and is_graph_embedding(g, h, f)
        }
        exists(f: V -> W) {
            is_graph_embedding(g, h, f)
        }
        contains_induced(h, g) = exists(f: V -> W) {
            is_graph_embedding(g, h, f)
        }
        contains_induced(h, g)
    }
}

/// An induced copy yields an embedding.
theorem contains_induced_witness[V, W](h: SimpleGraph[W], g: SimpleGraph[V]) {
    contains_induced(h, g) implies exists(f: V -> W) { is_graph_embedding(g, h, f) }
} by {
    if contains_induced(h, g) {
        contains_induced(h, g) = exists(f: V -> W) {
            is_graph_embedding(g, h, f)
        }
        exists(f: V -> W) {
            is_graph_embedding(g, h, f)
        }
    }
}

/// True if `h` has no induced copy of `g`.
define is_free_of[V, W](h: SimpleGraph[W], g: SimpleGraph[V]) -> Bool {
    not contains_induced(h, g)
}

/// A free graph admits no embedding of the forbidden one.
theorem is_free_of_apply[V, W](
    h: SimpleGraph[W], g: SimpleGraph[V], f: V -> W
) {
    is_free_of(h, g) implies not is_graph_embedding(g, h, f)
} by {
    if is_free_of(h, g) {
        is_free_of(h, g) = not contains_induced(h, g)
        not contains_induced(h, g)
        if is_graph_embedding(g, h, f) {
            contains_induced_intro(h, g, f)
            contains_induced(h, g)
            false
        }
        not is_graph_embedding(g, h, f)
    }
}

/// Admitting no embedding is freeness.
theorem is_free_of_intro[V, W](h: SimpleGraph[W], g: SimpleGraph[V]) {
    (forall(f: V -> W) { not is_graph_embedding(g, h, f) }) implies is_free_of(h, g)
} by {
    if forall(f: V -> W) { not is_graph_embedding(g, h, f) } {
        if contains_induced(h, g) {
            contains_induced_witness(h, g)
            let (e: V -> W) satisfy {
                is_graph_embedding(g, h, e)
            }
            not is_graph_embedding(g, h, e)
            false
        }
        not contains_induced(h, g)
        is_free_of(h, g) = not contains_induced(h, g)
        is_free_of(h, g)
    }
}

/// Freeness passes to anything embedded in a free graph.
///
/// A copy of the forbidden graph inside `k` would compose with the embedding of `k` into `h`
/// to give one inside `h`.
theorem is_free_of_embedded[U, V, W](
    h: SimpleGraph[W], k: SimpleGraph[V], g: SimpleGraph[U], f: V -> W
) {
    is_free_of(h, g) and is_graph_embedding(k, h, f) implies is_free_of(k, g)
} by {
    if is_free_of(h, g) and is_graph_embedding(k, h, f) {
        forall(e: U -> V) {
            if is_graph_embedding(g, k, e) {
                is_graph_embedding_compose(g, k, h, e, f)
                is_graph_embedding(g, h, compose(f, e))
                is_free_of_apply(h, g, compose(f, e))
                not is_graph_embedding(g, h, compose(f, e))
                false
            }
            not is_graph_embedding(g, k, e)
        }
        is_free_of_intro(k, g)
        is_free_of(k, g)
    }
}

/// An isomorphism is an embedding.
///
/// So a graph isomorphic to a free one is itself free, by the previous theorem.
theorem is_graph_iso_pair_is_embedding[V, W](
    g: SimpleGraph[V], h: SimpleGraph[W], f: V -> W, e: W -> V
) {
    is_graph_iso_pair(g, h, f, e) implies is_graph_embedding(g, h, f)
} by {
    if is_graph_iso_pair(g, h, f, e) {
        is_graph_iso_pair(g, h, f, e) =
            (is_graph_hom(g, h, f) and is_graph_hom(h, g, e)
                and are_inverse_vertex_maps(f, e))
        is_graph_hom(g, h, f)
        is_graph_iso_pair_reflects_adj(g, h, f, e)
        reflects_graph_adj(g, h, f)
        is_graph_iso_pair_map_is_injective(g, h, f, e)
        is_injective_fn(f)
        is_graph_embedding_intro(g, h, f)
        is_graph_embedding(g, h, f)
    }
}

/// Freeness transports across an isomorphism.
theorem is_free_of_iso[U, V, W](
    h: SimpleGraph[W], k: SimpleGraph[V], g: SimpleGraph[U], f: V -> W, e: W -> V
) {
    is_free_of(h, g) and is_graph_iso_pair(k, h, f, e) implies is_free_of(k, g)
} by {
    if is_free_of(h, g) and is_graph_iso_pair(k, h, f, e) {
        is_graph_iso_pair_is_embedding(k, h, f, e)
        is_graph_embedding(k, h, f)
        is_free_of_embedded(h, k, g, f)
        is_free_of(k, g)
    }
}
