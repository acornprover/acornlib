/// The chromatic number of a graph: the least number of colors of a proper vertex coloring.

from nat import Nat, lt_trans, lt_and_lte, lte_and_lt, lt_cancel_suc, not_lt_zero,
    lt_suc_right, lte_antisymm, lt_or_lte, lt_not_ref, lt_imp_lte_suc, lte_cancel_suc, lt_suc,
    small_mod, div_imp_mod, divides_self, alt_induction, sub_one_lt, add_imp_sub,
    alt_suc_ne_zero, is_min, has_min, is_min_apply, is_min_false_below, false_below,
    false_below_apply
from list import List, range_pigeonhole, map, map_contains, map_length, map_add, map_singleton,
    add_length, add_contains_or, singleton_contains_imp_eq, unique_implies_tail_unique,
    unique_is_smallest_containing_list, length_range, range_contains_iff_lt, range_is_unique
from data.list.list_cons_membership import cons_contains_eq, cons_contains_head,
    cons_contains_of_tail_contains
from data.basic.functions import is_injective_fn
from finite_set import FiniteSet, fs_from_list, finite_set_from_unique_list_cardinality_is_length,
    finite_set_cardinality_has_exact_unique_list
from data.finite.finite_set_membership import fs_from_list_contains_eq
from data.finite.finite_set_card import fs_card, fs_card_eq_of_cardinality_is,
    fs_card_cardinality_is
from graph.simple_graph import SimpleGraph, complete_graph, complete_graph_adj_iff_ne,
    simple_graph_adj_ne
from graph.simple_graph_coloring import simple_graph_coloring
from graph.simple_graph_bipartite import is_bipartite, is_bipartite_intro,
    is_bipartite_witness, is_bipartition, is_bipartition_apply, is_bipartition_intro
from graph.simple_graph_bipartite_odd_cycle import nat_odd, nat_odd_zero, nat_odd_suc,
    walk_in_set, walk_in_set_nil, walk_in_set_cons, walk_in_set_cons_head,
    walk_in_set_cons_tail, bipartition_walk_color_flip
from graph.simple_graph_cycle import cycle_graph, cycle_graph_adj_iff, cycle_graph_adj_suc,
    cycle_graph_adj_wrap
from graph.simple_graph_walks import simple_graph_walk, simple_graph_walk_nil,
    simple_graph_walk_append_adj, simple_graph_closed_walk, simple_graph_closed_walk_intro,
    simple_graph_closed_walk_is_walk

numerals Nat

/// A proper coloring of the vertices of `s`: adjacent pairs inside `s` receive distinct colors.
///
/// This is the finite-vertex-set version of `simple_graph_coloring`: the graph predicates of
/// this library carry their vertex set separately, so the coloring condition is imposed only on
/// pairs of vertices of `s`. A global proper coloring restricts to one here.
define is_proper_coloring_on[V, C](g: SimpleGraph[V], s: FiniteSet[V], color: V -> C) -> Bool {
    forall(x: V, y: V) {
        s.contains(x) and s.contains(y) and g.adj(x, y) implies color(x) != color(y)
    }
}

/// Adjacent vertices of `s` receive distinct colors from a proper coloring on `s`.
theorem is_proper_coloring_on_apply[V, C](
    g: SimpleGraph[V], s: FiniteSet[V], color: V -> C, x: V, y: V
) {
    is_proper_coloring_on(g, s, color) and s.contains(x) and s.contains(y) and g.adj(x, y)
        implies color(x) != color(y)
} by {
    if is_proper_coloring_on(g, s, color) and s.contains(x) and s.contains(y) and g.adj(x, y) {
        is_proper_coloring_on(g, s, color) = forall(a: V, b: V) {
            s.contains(a) and s.contains(b) and g.adj(a, b) implies color(a) != color(b)
        }
        forall(a: V, b: V) {
            s.contains(a) and s.contains(b) and g.adj(a, b) implies color(a) != color(b)
        }
        s.contains(x) and s.contains(y) and g.adj(x, y) implies color(x) != color(y)
        color(x) != color(y)
    }
}

/// A pointwise distinct-colors condition on `s` is a proper coloring on `s`.
theorem is_proper_coloring_on_intro[V, C](g: SimpleGraph[V], s: FiniteSet[V], color: V -> C) {
    (forall(x: V, y: V) {
        s.contains(x) and s.contains(y) and g.adj(x, y) implies color(x) != color(y)
    }) implies is_proper_coloring_on(g, s, color)
} by {
    if forall(x: V, y: V) {
        s.contains(x) and s.contains(y) and g.adj(x, y) implies color(x) != color(y)
    } {
        is_proper_coloring_on(g, s, color) = forall(a: V, b: V) {
            s.contains(a) and s.contains(b) and g.adj(a, b) implies color(a) != color(b)
        }
        is_proper_coloring_on(g, s, color)
    }
}

/// A global proper coloring restricts to any finite vertex set.
theorem is_proper_coloring_on_of_simple_graph_coloring[V, C](
    g: SimpleGraph[V], s: FiniteSet[V], color: V -> C
) {
    simple_graph_coloring(g, color) implies is_proper_coloring_on(g, s, color)
} by {
    if simple_graph_coloring(g, color) {
        forall(x: V, y: V) {
            if s.contains(x) and s.contains(y) and g.adj(x, y) {
                simple_graph_coloring(g, color) = forall(a: V, b: V) {
                    g.adj(a, b) implies color(a) != color(b)
                }
                forall(a: V, b: V) {
                    g.adj(a, b) implies color(a) != color(b)
                }
                g.adj(x, y) implies color(x) != color(y)
                color(x) != color(y)
            }
            s.contains(x) and s.contains(y) and g.adj(x, y) implies color(x) != color(y)
        }
        is_proper_coloring_on_intro(g, s, color)
        is_proper_coloring_on(g, s, color)
    }
}

/// True if `g` has a proper coloring of `s` using at most `k` colors.
///
/// A `k`-coloring is a map from vertices into the natural numbers taking values below `k` on
/// `s`, so that `k` is the size of the palette.
define is_k_colorable[V](g: SimpleGraph[V], s: FiniteSet[V], k: Nat) -> Bool {
    exists(color: V -> Nat) {
        is_proper_coloring_on(g, s, color) and forall(v: V) {
            s.contains(v) implies color(v) < k
        }
    }
}

/// A proper coloring with values below `k` on `s` witnesses `k`-colorability.
theorem is_k_colorable_intro[V](g: SimpleGraph[V], s: FiniteSet[V], k: Nat, color: V -> Nat) {
    is_proper_coloring_on(g, s, color) and forall(v: V) { s.contains(v) implies color(v) < k }
        implies is_k_colorable(g, s, k)
} by {
    if is_proper_coloring_on(g, s, color) and forall(v: V) { s.contains(v) implies color(v) < k } {
        is_k_colorable(g, s, k) = exists(other: V -> Nat) {
            is_proper_coloring_on(g, s, other) and forall(v: V) {
                s.contains(v) implies other(v) < k
            }
        }
        exists(other: V -> Nat) {
            is_proper_coloring_on(g, s, other) and forall(v: V) {
                s.contains(v) implies other(v) < k
            }
        }
        is_k_colorable(g, s, k)
    }
}

/// A `k`-coloring provides a proper coloring and a bound on its values.
theorem is_k_colorable_witness[V](g: SimpleGraph[V], s: FiniteSet[V], k: Nat) {
    is_k_colorable(g, s, k) implies exists(color: V -> Nat) {
        is_proper_coloring_on(g, s, color) and forall(v: V) {
            s.contains(v) implies color(v) < k
        }
    }
} by {
    if is_k_colorable(g, s, k) {
        is_k_colorable(g, s, k) = exists(other: V -> Nat) {
            is_proper_coloring_on(g, s, other) and forall(v: V) {
                s.contains(v) implies other(v) < k
            }
        }
        exists(other: V -> Nat) {
            is_proper_coloring_on(g, s, other) and forall(v: V) {
                s.contains(v) implies other(v) < k
            }
        }
    }
}

/// Enlarging the palette never destroys a coloring: `k`-colorability is monotone in `k`.
theorem is_k_colorable_mono[V](g: SimpleGraph[V], s: FiniteSet[V], k1: Nat, k2: Nat) {
    is_k_colorable(g, s, k1) and k1 <= k2 implies is_k_colorable(g, s, k2)
} by {
    if is_k_colorable(g, s, k1) and k1 <= k2 {
        is_k_colorable_witness(g, s, k1)
        let (color: V -> Nat) satisfy {
            is_proper_coloring_on(g, s, color) and forall(v: V) {
                s.contains(v) implies color(v) < k1
            }
        }
        is_proper_coloring_on(g, s, color)
        forall(v: V) {
            if s.contains(v) {
                color(v) < k1
                lt_and_lte(color(v), k1, k2)
                color(v) < k2
            }
            s.contains(v) implies color(v) < k2
        }
        is_k_colorable_intro(g, s, k2, color)
        is_k_colorable(g, s, k2)
    }
}

/// A natural below two is zero or one.
theorem nat_lt_two_cases(n: Nat) {
    n < Nat.2 implies n = Nat.0 or n = Nat.1
} by {
    if n < Nat.2 {
        match n {
            Nat.zero {
                n = Nat.0 or n = Nat.1
            }
            Nat.suc(pred) {
                lt_cancel_suc(pred, Nat.1)
                pred < Nat.1
                match pred {
                    Nat.zero {
                        n = Nat.1
                        n = Nat.0 or n = Nat.1
                    }
                    Nat.suc(k) {
                        lt_cancel_suc(k, Nat.0)
                        k < Nat.0
                        not_lt_zero(k)
                        false
                    }
                }
            }
        }
    }
}

/// A natural below two that is not zero is one.
theorem nat_lt_two_one_of_ne_zero(n: Nat) {
    n < Nat.2 and n != Nat.0 implies n = Nat.1
} by {
    if n < Nat.2 and n != Nat.0 {
        nat_lt_two_cases(n)
        n = Nat.0 or n = Nat.1
        if n = Nat.0 {
            n != Nat.0
            false
        }
        n = Nat.1
    }
}

/// Distinct naturals below two differ in whether they equal one.
theorem two_color_ne_imp_one_eq_ne(a: Nat, b: Nat) {
    a < Nat.2 and b < Nat.2 and a != b implies (a = Nat.1) != (b = Nat.1)
} by {
    if a < Nat.2 and b < Nat.2 and a != b {
        if a = Nat.1 {
            a != b
            if b = Nat.1 {
                a = b
                false
            }
            not (b = Nat.1)
            if (a = Nat.1) = (b = Nat.1) {
                (a = Nat.1)
                (b = Nat.1)
                false
            }
            (a = Nat.1) != (b = Nat.1)
        }
        if a != Nat.1 {
            nat_lt_two_one_of_ne_zero(a)
            a = Nat.0
            a != b
            if b = Nat.0 {
                a = b
                false
            }
            not (b = Nat.0)
            nat_lt_two_one_of_ne_zero(b)
            b = Nat.1
            if (a = Nat.1) = (b = Nat.1) {
                (b = Nat.1)
                (a = Nat.1)
                false
            }
            (a = Nat.1) != (b = Nat.1)
        }
        (a = Nat.1) != (b = Nat.1)
    }
}

/// The boolean side of a two-coloring: true when the vertex carries color one.
define part_of_two_color[V](color: V -> Nat) -> (V -> Bool) {
    function(v: V) {
        color(v) = Nat.1
    }
}

/// The side of a two-coloring is the test of whether the color is one.
theorem part_of_two_color_eq[V](color: V -> Nat, v: V) {
    part_of_two_color(color)(v) = (color(v) = Nat.1)
}

/// The natural coloring of a two-coloring: color one for true, zero for false.
define two_color_of_part[V](part: V -> Bool) -> (V -> Nat) {
    function(v: V) {
        if part(v) {
            Nat.1
        } else {
            Nat.0
        }
    }
}

/// The color of a vertex is one exactly on the true side.
theorem two_color_of_part_eq[V](part: V -> Bool, v: V) {
    two_color_of_part(part)(v) = if part(v) { Nat.1 } else { Nat.0 }
}

/// A two-coloring of a graph is a bipartition: the true side of the coloring separates every edge.
theorem is_k_colorable_two_imp_is_bipartite[V](g: SimpleGraph[V], s: FiniteSet[V]) {
    is_k_colorable(g, s, Nat.2) implies is_bipartite(g, s)
} by {
    if is_k_colorable(g, s, Nat.2) {
        is_k_colorable_witness(g, s, Nat.2)
        let (color: V -> Nat) satisfy {
            is_proper_coloring_on(g, s, color) and forall(v: V) {
                s.contains(v) implies color(v) < Nat.2
            }
        }
        is_bipartition_intro(g, s, part_of_two_color(color))
        forall(x: V, y: V) {
            if s.contains(x) and s.contains(y) and g.adj(x, y) {
                is_proper_coloring_on_apply(g, s, color, x, y)
                color(x) != color(y)
                color(x) < Nat.2
                color(y) < Nat.2
                two_color_ne_imp_one_eq_ne(color(x), color(y))
                (color(x) = Nat.1) != (color(y) = Nat.1)
                part_of_two_color_eq(color, x)
                part_of_two_color(color)(x) = (color(x) = Nat.1)
                part_of_two_color_eq(color, y)
                part_of_two_color(color)(y) = (color(y) = Nat.1)
                part_of_two_color(color)(x) != part_of_two_color(color)(y)
            }
            s.contains(x) and s.contains(y) and g.adj(x, y) implies part_of_two_color(color)(x) != part_of_two_color(color)(y)
        }
        is_bipartition(g, s, part_of_two_color(color))
        is_bipartite_intro(g, s, part_of_two_color(color))
        is_bipartite(g, s)
    }
}

/// A bipartition is a two-coloring: true side gets color one, false side color zero.
theorem is_bipartite_imp_is_k_colorable_two[V](g: SimpleGraph[V], s: FiniteSet[V]) {
    is_bipartite(g, s) implies is_k_colorable(g, s, Nat.2)
} by {
    if is_bipartite(g, s) {
        is_bipartite_witness(g, s)
        let (part: V -> Bool) satisfy {
            is_bipartition(g, s, part)
        }
        is_k_colorable_intro(g, s, Nat.2, two_color_of_part(part))
        is_proper_coloring_on_intro(g, s, two_color_of_part(part))
        forall(x: V, y: V) {
            if s.contains(x) and s.contains(y) and g.adj(x, y) {
                is_bipartition_apply(g, s, part, x, y)
                part(x) != part(y)
                if part(x) {
                    if part(y) {
                        part(x) = part(y)
                        false
                    }
                    not part(y)
                    two_color_of_part_eq(part, x)
                    two_color_of_part(part)(x) = if part(x) { Nat.1 } else { Nat.0 }
                    two_color_of_part(part)(x) = Nat.1
                    two_color_of_part_eq(part, y)
                    two_color_of_part(part)(y) = if part(y) { Nat.1 } else { Nat.0 }
                    two_color_of_part(part)(y) = Nat.0
                    Nat.1 != Nat.0
                    two_color_of_part(part)(x) != two_color_of_part(part)(y)
                }
                if not part(x) {
                    if not part(y) {
                        part(x) = part(y)
                        false
                    }
                    part(y)
                    two_color_of_part_eq(part, x)
                    two_color_of_part(part)(x) = if part(x) { Nat.1 } else { Nat.0 }
                    two_color_of_part(part)(x) = Nat.0
                    two_color_of_part_eq(part, y)
                    two_color_of_part(part)(y) = if part(y) { Nat.1 } else { Nat.0 }
                    two_color_of_part(part)(y) = Nat.1
                    Nat.1 != Nat.0
                    two_color_of_part(part)(x) != two_color_of_part(part)(y)
                }
                two_color_of_part(part)(x) != two_color_of_part(part)(y)
            }
            s.contains(x) and s.contains(y) and g.adj(x, y) implies two_color_of_part(part)(x) != two_color_of_part(part)(y)
        }
        forall(v: V) {
            if s.contains(v) {
                if part(v) {
                    two_color_of_part_eq(part, v)
                    two_color_of_part(part)(v) = if part(v) { Nat.1 } else { Nat.0 }
                    two_color_of_part(part)(v) = Nat.1
                    Nat.1 < Nat.2
                    two_color_of_part(part)(v) < Nat.2
                }
                if not part(v) {
                    two_color_of_part_eq(part, v)
                    two_color_of_part(part)(v) = if part(v) { Nat.1 } else { Nat.0 }
                    two_color_of_part(part)(v) = Nat.0
                    Nat.0 < Nat.2
                    two_color_of_part(part)(v) < Nat.2
                }
                two_color_of_part(part)(v) < Nat.2
            }
            s.contains(v) implies two_color_of_part(part)(v) < Nat.2
        }
        is_proper_coloring_on(g, s, two_color_of_part(part))
        forall(v: V) { s.contains(v) implies two_color_of_part(part)(v) < Nat.2 }
        is_k_colorable(g, s, Nat.2)
    }
}

/// A graph is two-colorable exactly when it is bipartite.
theorem is_k_colorable_two_eq_is_bipartite[V](g: SimpleGraph[V], s: FiniteSet[V]) {
    is_k_colorable(g, s, Nat.2) = is_bipartite(g, s)
} by {
    is_k_colorable_two_imp_is_bipartite(g, s)
    is_bipartite_imp_is_k_colorable_two(g, s)
    (is_k_colorable(g, s, Nat.2) implies is_bipartite(g, s))
    (is_bipartite(g, s) implies is_k_colorable(g, s, Nat.2))
    is_k_colorable(g, s, Nat.2) = is_bipartite(g, s)
}

/// True if `k` is the chromatic number of `g` on the vertex set `s`: `k` colors suffice and
/// no smaller number does.
define is_chromatic_number[V](g: SimpleGraph[V], s: FiniteSet[V], k: Nat) -> Bool {
    is_k_colorable(g, s, k) and forall(j: Nat) {
        j < k implies not is_k_colorable(g, s, j)
    }
}

/// The chromatic number admits a coloring and rejects every smaller palette.
theorem is_chromatic_number_apply[V](g: SimpleGraph[V], s: FiniteSet[V], k: Nat) {
    is_chromatic_number(g, s, k) implies
        is_k_colorable(g, s, k) and forall(j: Nat) {
            j < k implies not is_k_colorable(g, s, j)
        }
} by {
    if is_chromatic_number(g, s, k) {
        is_chromatic_number(g, s, k) = (
            is_k_colorable(g, s, k) and forall(j: Nat) {
                j < k implies not is_k_colorable(g, s, j)
            }
        )
        is_k_colorable(g, s, k) and forall(j: Nat) {
            j < k implies not is_k_colorable(g, s, j)
        }
    }
}

/// A colorability that rejects every smaller palette is the chromatic number.
theorem is_chromatic_number_intro[V](g: SimpleGraph[V], s: FiniteSet[V], k: Nat) {
    is_k_colorable(g, s, k) and forall(j: Nat) {
        j < k implies not is_k_colorable(g, s, j)
    } implies is_chromatic_number(g, s, k)
} by {
    if is_k_colorable(g, s, k) and forall(j: Nat) {
        j < k implies not is_k_colorable(g, s, j)
    } {
        is_chromatic_number(g, s, k) = (
            is_k_colorable(g, s, k) and forall(j: Nat) {
                j < k implies not is_k_colorable(g, s, j)
            }
        )
        is_chromatic_number(g, s, k)
    }
}

/// The chromatic number is a colorability.
theorem is_chromatic_number_k_colorable[V](g: SimpleGraph[V], s: FiniteSet[V], k: Nat) {
    is_chromatic_number(g, s, k) implies is_k_colorable(g, s, k)
} by {
    if is_chromatic_number(g, s, k) {
        is_chromatic_number_apply(g, s, k)
        is_k_colorable(g, s, k) and forall(j: Nat) {
            j < k implies not is_k_colorable(g, s, j)
        }
        is_k_colorable(g, s, k)
    }
}

/// No palette below the chromatic number colors the graph.
theorem is_chromatic_number_not_k_colorable_below[V](
    g: SimpleGraph[V], s: FiniteSet[V], k: Nat, j: Nat
) {
    is_chromatic_number(g, s, k) and j < k implies not is_k_colorable(g, s, j)
} by {
    if is_chromatic_number(g, s, k) and j < k {
        is_chromatic_number_apply(g, s, k)
        is_k_colorable(g, s, k) and forall(j2: Nat) {
            j2 < k implies not is_k_colorable(g, s, j2)
        }
        forall(j2: Nat) { j2 < k implies not is_k_colorable(g, s, j2) }
        j < k implies not is_k_colorable(g, s, j)
        not is_k_colorable(g, s, j)
    }
}

/// The chromatic number is unique.
theorem is_chromatic_number_unique[V](g: SimpleGraph[V], s: FiniteSet[V], k1: Nat, k2: Nat) {
    is_chromatic_number(g, s, k1) and is_chromatic_number(g, s, k2) implies k1 = k2
} by {
    if is_chromatic_number(g, s, k1) and is_chromatic_number(g, s, k2) {
        if k2 < k1 {
            is_chromatic_number_not_k_colorable_below(g, s, k1, k2)
            not is_k_colorable(g, s, k2)
            is_chromatic_number_k_colorable(g, s, k2)
            is_k_colorable(g, s, k2)
            false
        }
        not (k2 < k1)
        lt_or_lte(k2, k1)
        k2 < k1 or k1 <= k2
        k1 <= k2
        if k1 < k2 {
            is_chromatic_number_not_k_colorable_below(g, s, k2, k1)
            not is_k_colorable(g, s, k1)
            is_chromatic_number_k_colorable(g, s, k1)
            is_k_colorable(g, s, k1)
            false
        }
        not (k1 < k2)
        lt_or_lte(k1, k2)
        k1 < k2 or k2 <= k1
        k2 <= k1
        lte_antisymm(k1, k2)
        k1 = k2
    }
}

/// The finite vertex set `{0, ..., n - 1}`, the standard vertex set of the complete graph and
/// the cycle.
define nat_range_set(n: Nat) -> FiniteSet[Nat] {
    fs_from_list(n.range)
}

/// Membership in the standard vertex set is being below `n`.
theorem nat_range_set_contains_eq(n: Nat, x: Nat) {
    nat_range_set(n).contains(x) = (x < n)
} by {
    nat_range_set(n) = fs_from_list(n.range)
    nat_range_set(n).contains(x) = fs_from_list(n.range).contains(x)
    fs_from_list_contains_eq(n.range, x)
    fs_from_list(n.range).contains(x) = n.range.contains(x)
    range_contains_iff_lt(n, x)
    n.range.contains(x) = (x < n)
    nat_range_set(n).contains(x) = (x < n)
}

/// The standard vertex set has `n` elements.
theorem nat_range_set_card(n: Nat) {
    fs_card(nat_range_set(n)) = n
} by {
    range_is_unique(n)
    n.range.is_unique
    finite_set_from_unique_list_cardinality_is_length(n.range)
    fs_from_list(n.range).cardinality_is(n.range.length)
    length_range(n)
    n.range.length = n
    fs_from_list(n.range).cardinality_is(n)
    nat_range_set(n) = fs_from_list(n.range)
    nat_range_set(n).cardinality_is(n)
    fs_card_eq_of_cardinality_is(nat_range_set(n), n)
    fs_card(nat_range_set(n)) = n
}

/// The identity map on the natural numbers.
define nat_identity(v: Nat) -> Nat { v }

/// The identity map takes a natural to itself.
theorem nat_identity_eq(v: Nat) {
    nat_identity(v) = v
}

/// The unique image list is no longer than any list containing the image.
theorem map_unique_length_le_of_contains[T, U](items: List[T], targets: List[U], f: T -> U) {
    forall(x: T) {
        items.contains(x) implies targets.contains(f(x))
    } implies map(items, f).unique.length <= targets.length
} by {
    if forall(x: T) {
        items.contains(x) implies targets.contains(f(x))
    } {
        forall(y: U) {
            if map(items, f).contains(y) {
                map_contains(items, f, y)
                let (x: T) satisfy {
                    items.contains(x) and f(x) = y
                }
                items.contains(x) implies targets.contains(f(x))
                targets.contains(f(x))
                targets.contains(y)
            }
            map(items, f).contains(y) implies targets.contains(y)
        }
        unique_is_smallest_containing_list(map(items, f), targets)
        map(items, f).unique.length <= targets.length
    }
}

/// The complete graph on the standard `n`-vertex set is `n`-colorable: the identity is a proper
/// coloring.
theorem complete_graph_nat_range_is_k_colorable(n: Nat) {
    is_k_colorable(complete_graph[Nat], nat_range_set(n), n)
} by {
    is_k_colorable_intro(complete_graph[Nat], nat_range_set(n), n, nat_identity)
    is_proper_coloring_on_intro(complete_graph[Nat], nat_range_set(n), nat_identity)
    forall(x: Nat, y: Nat) {
        if nat_range_set(n).contains(x) and nat_range_set(n).contains(y) and
            complete_graph[Nat].adj(x, y) {
            complete_graph_adj_iff_ne[Nat](x, y)
            x != y
            nat_identity_eq(x)
            nat_identity(x) = x
            nat_identity_eq(y)
            nat_identity(y) = y
            nat_identity(x) != nat_identity(y)
        }
        nat_range_set(n).contains(x) and nat_range_set(n).contains(y) and
            complete_graph[Nat].adj(x, y) implies nat_identity(x) != nat_identity(y)
    }
    forall(v: Nat) {
        if nat_range_set(n).contains(v) {
            nat_range_set_contains_eq(n, v)
            v < n
            nat_identity_eq(v)
            nat_identity(v) = v
            nat_identity(v) < n
        }
        nat_range_set(n).contains(v) implies nat_identity(v) < n
    }
    is_proper_coloring_on(complete_graph[Nat], nat_range_set(n), nat_identity)
    forall(v: Nat) { nat_range_set(n).contains(v) implies nat_identity(v) < n }
    is_k_colorable(complete_graph[Nat], nat_range_set(n), n)
}

/// Fewer than `n` colors cannot color the complete graph on the standard `n`-vertex set.
///
/// A proper coloring of the complete graph is injective on the vertices, and an injective map
/// of `n` vertices into a palette of `k < n` colors contradicts the pigeonhole principle.
theorem complete_graph_nat_range_not_k_colorable_below(n: Nat, k: Nat) {
    k < n implies not is_k_colorable(complete_graph[Nat], nat_range_set(n), k)
} by {
    if k < n {
        if is_k_colorable(complete_graph[Nat], nat_range_set(n), k) {
            is_k_colorable_witness(complete_graph[Nat], nat_range_set(n), k)
            let (color: Nat -> Nat) satisfy {
                is_proper_coloring_on(complete_graph[Nat], nat_range_set(n), color) and
                    forall(v: Nat) {
                        nat_range_set(n).contains(v) implies color(v) < k
                    }
            }
            forall(x: Nat) {
                if n.range.contains(x) {
                    range_contains_iff_lt(n, x)
                    x < n
                    nat_range_set_contains_eq(n, x)
                    nat_range_set(n).contains(x)
                    color(x) < k
                    range_contains_iff_lt(k, color(x))
                    k.range.contains(color(x))
                }
                n.range.contains(x) implies k.range.contains(color(x))
            }
            map_unique_length_le_of_contains(n.range, k.range, color)
            map(n.range, color).unique.length <= k.range.length
            length_range(k)
            k.range.length = k
            map(n.range, color).unique.length <= k
            lte_and_lt(map(n.range, color).unique.length, k, n)
            map(n.range, color).unique.length < n
            range_pigeonhole(n, color)
            exists(i: Nat, j: Nat) { i < j and j < n and color(i) = color(j) }
            let (i: Nat, j: Nat) satisfy { i < j and j < n and color(i) = color(j) }
            i < j
            j < n
            color(i) = color(j)
            lt_trans(i, j, n)
            i < n
            nat_range_set_contains_eq(n, i)
            nat_range_set(n).contains(i)
            nat_range_set_contains_eq(n, j)
            nat_range_set(n).contains(j)
            lt_not_ref(i)
            if i = j {
                i < i
                false
            }
            i != j
            complete_graph_adj_iff_ne[Nat](i, j)
            complete_graph[Nat].adj(i, j)
            is_proper_coloring_on_apply(complete_graph[Nat], nat_range_set(n), color, i, j)
            color(i) != color(j)
            false
        }
        not is_k_colorable(complete_graph[Nat], nat_range_set(n), k)
    }
}

/// The chromatic number of the complete graph on `n` vertices is `n`.
theorem complete_graph_nat_range_chromatic_number(n: Nat) {
    is_chromatic_number(complete_graph[Nat], nat_range_set(n), n)
} by {
    is_chromatic_number_intro(complete_graph[Nat], nat_range_set(n), n)
    complete_graph_nat_range_is_k_colorable(n)
    is_k_colorable(complete_graph[Nat], nat_range_set(n), n)
    forall(j: Nat) {
        if j < n {
            complete_graph_nat_range_not_k_colorable_below(n, j)
            not is_k_colorable(complete_graph[Nat], nat_range_set(n), j)
        }
        j < n implies not is_k_colorable(complete_graph[Nat], nat_range_set(n), j)
    }
    is_chromatic_number(complete_graph[Nat], nat_range_set(n), n)
}

// ============================================================================
// Cycles: chi(C_n) = 2 for even n, chi(C_n) = 3 for odd n.
// ============================================================================

/// A boolean and its negation differ.
theorem bool_ne_not_self(x: Bool) {
    x != not x
} by {
    if x {
        if x = not x {
            x
            not x
            false
        }
        x != not x
    }
    if not x {
        if x = not x {
            not x
            x
            false
        }
        x != not x
    }
    x != not x
}

/// A forward edge of an even cycle joins vertices of different parities.
theorem cycle_graph_even_suc_parity_ne(n: Nat, x: Nat, y: Nat) {
    not nat_odd(n) and x < n and y < n and x.suc.mod(n) = y
        implies nat_odd(x) != nat_odd(y)
} by {
    if not nat_odd(n) and x < n and y < n and x.suc.mod(n) = y {
        if x.suc < n {
            small_mod(x.suc, n)
            x.suc.mod(n) = x.suc
            y = x.suc
            nat_odd_suc(x)
            nat_odd(x.suc) = not nat_odd(x)
            nat_odd(y) = not nat_odd(x)
            bool_ne_not_self(nat_odd(x))
            nat_odd(x) != not nat_odd(x)
            nat_odd(x) != nat_odd(y)
        }
        if not (x.suc < n) {
            lt_imp_lte_suc(x, n)
            x.suc <= n
            lt_or_lte(x.suc, n)
            x.suc < n or n <= x.suc
            n <= x.suc
            lte_antisymm(x.suc, n)
            x.suc = n
            divides_self(n)
            div_imp_mod(n, n)
            n.mod(n) = Nat.0
            x.suc.mod(n) = n.mod(n)
            x.suc.mod(n) = Nat.0
            y = Nat.0
            nat_odd_zero
            nat_odd(Nat.0) = false
            nat_odd(y) = false
            nat_odd_suc(x)
            nat_odd(x.suc) = not nat_odd(x)
            nat_odd(x.suc) = nat_odd(n)
            nat_odd(n) = not nat_odd(x)
            if nat_odd(x) {
                nat_odd(x)
            }
            if not nat_odd(x) {
                nat_odd(n) = not nat_odd(x)
                nat_odd(n)
                not nat_odd(n)
                false
            }
            nat_odd(x)
            if nat_odd(x) = nat_odd(y) {
                nat_odd(y)
                false
            }
            nat_odd(x) != nat_odd(y)
        }
        nat_odd(x) != nat_odd(y)
    }
}

/// Adjacent vertices of an even cycle have different parities.
theorem cycle_graph_even_adj_parity_ne(n: Nat, x: Nat, y: Nat) {
    not nat_odd(n) and x < n and y < n and cycle_graph(n).adj(x, y)
        implies nat_odd(x) != nat_odd(y)
} by {
    if not nat_odd(n) and x < n and y < n and cycle_graph(n).adj(x, y) {
        cycle_graph_adj_iff(n, x, y)
        x.suc.mod(n) = y or y.suc.mod(n) = x
        if x.suc.mod(n) = y {
            cycle_graph_even_suc_parity_ne(n, x, y)
            nat_odd(x) != nat_odd(y)
        }
        if y.suc.mod(n) = x {
            cycle_graph_even_suc_parity_ne(n, y, x)
            nat_odd(y) != nat_odd(x)
            nat_odd(x) != nat_odd(y)
        }
        nat_odd(x) != nat_odd(y)
    }
}

/// The parity function is a bipartition of an even cycle.
theorem cycle_graph_even_bipartition(n: Nat) {
    not nat_odd(n) implies is_bipartition(cycle_graph(n), nat_range_set(n), nat_odd)
} by {
    if not nat_odd(n) {
        is_bipartition_intro(cycle_graph(n), nat_range_set(n), nat_odd)
        forall(x: Nat, y: Nat) {
            if nat_range_set(n).contains(x) and nat_range_set(n).contains(y) and
                cycle_graph(n).adj(x, y) {
                nat_range_set_contains_eq(n, x)
                x < n
                nat_range_set_contains_eq(n, y)
                y < n
                cycle_graph_even_adj_parity_ne(n, x, y)
                nat_odd(x) != nat_odd(y)
            }
            nat_range_set(n).contains(x) and nat_range_set(n).contains(y) and
                cycle_graph(n).adj(x, y) implies nat_odd(x) != nat_odd(y)
        }
        is_bipartition(cycle_graph(n), nat_range_set(n), nat_odd)
    }
}

/// An even cycle is two-colorable.
theorem cycle_graph_even_is_k_colorable_two(n: Nat) {
    not nat_odd(n) implies is_k_colorable(cycle_graph(n), nat_range_set(n), Nat.2)
} by {
    if not nat_odd(n) {
        cycle_graph_even_bipartition(n)
        is_bipartition(cycle_graph(n), nat_range_set(n), nat_odd)
        is_bipartite_intro(cycle_graph(n), nat_range_set(n), nat_odd)
        is_bipartite(cycle_graph(n), nat_range_set(n))
        is_bipartite_imp_is_k_colorable_two(cycle_graph(n), nat_range_set(n))
        is_k_colorable(cycle_graph(n), nat_range_set(n), Nat.2)
    }
}

/// A natural below one is zero.
theorem nat_lt_one_eq_zero(n: Nat) {
    n < Nat.1 implies n = Nat.0
} by {
    if n < Nat.1 {
        lt_suc_right(n, Nat.0)
        n = Nat.0 or n < Nat.0
        if n < Nat.0 {
            not_lt_zero(n)
            false
        }
        n = Nat.0
    }
}

/// The empty palette colors nothing on a nonempty vertex set.
theorem not_is_k_colorable_zero[V](g: SimpleGraph[V], s: FiniteSet[V], v: V) {
    s.contains(v) implies not is_k_colorable(g, s, Nat.0)
} by {
    if s.contains(v) {
        if is_k_colorable(g, s, Nat.0) {
            is_k_colorable_witness(g, s, Nat.0)
            let (color: V -> Nat) satisfy {
                is_proper_coloring_on(g, s, color) and forall(v2: V) {
                    s.contains(v2) implies color(v2) < Nat.0
                }
            }
            s.contains(v) implies color(v) < Nat.0
            color(v) < Nat.0
            not_lt_zero(color(v))
            false
        }
        not is_k_colorable(g, s, Nat.0)
    }
}

/// A graph with an edge is not one-colorable.
theorem not_is_k_colorable_one_of_adj[V](g: SimpleGraph[V], s: FiniteSet[V], x: V, y: V) {
    s.contains(x) and s.contains(y) and g.adj(x, y) implies not is_k_colorable(g, s, Nat.1)
} by {
    if s.contains(x) and s.contains(y) and g.adj(x, y) {
        if is_k_colorable(g, s, Nat.1) {
            is_k_colorable_witness(g, s, Nat.1)
            let (color: V -> Nat) satisfy {
                is_proper_coloring_on(g, s, color) and forall(v: V) {
                    s.contains(v) implies color(v) < Nat.1
                }
            }
            is_proper_coloring_on_apply(g, s, color, x, y)
            color(x) != color(y)
            color(x) < Nat.1
            nat_lt_one_eq_zero(color(x))
            color(x) = Nat.0
            color(y) < Nat.1
            nat_lt_one_eq_zero(color(y))
            color(y) = Nat.0
            color(x) = color(y)
            false
        }
        not is_k_colorable(g, s, Nat.1)
    }
}

/// The chromatic number of an even cycle is two.
theorem cycle_graph_even_chromatic_number(n: Nat) {
    Nat.3 <= n and not nat_odd(n) implies
        is_chromatic_number(cycle_graph(n), nat_range_set(n), Nat.2)
} by {
    if Nat.3 <= n and not nat_odd(n) {
        is_chromatic_number_intro(cycle_graph(n), nat_range_set(n), Nat.2)
        cycle_graph_even_is_k_colorable_two(n)
        is_k_colorable(cycle_graph(n), nat_range_set(n), Nat.2)
        forall(j: Nat) {
            if j < Nat.2 {
                nat_lt_two_cases(j)
                j = Nat.0 or j = Nat.1
                if j = Nat.0 {
                    Nat.0 < Nat.3
                    lt_and_lte(Nat.0, Nat.3, n)
                    Nat.0 < n
                    nat_range_set_contains_eq(n, Nat.0)
                    nat_range_set(n).contains(Nat.0)
                    not_is_k_colorable_zero(cycle_graph(n), nat_range_set(n), Nat.0)
                    not is_k_colorable(cycle_graph(n), nat_range_set(n), Nat.0)
                    not is_k_colorable(cycle_graph(n), nat_range_set(n), j)
                }
                if j = Nat.1 {
                    Nat.1 < Nat.3
                    lt_and_lte(Nat.1, Nat.3, n)
                    Nat.1 < n
                    lt_suc(Nat.0)
                    Nat.0 < Nat.1
                    lt_trans(Nat.0, Nat.1, n)
                    Nat.0 < n
                    cycle_graph_adj_suc(n, Nat.0)
                    cycle_graph(n).adj(Nat.0, Nat.0.suc)
                    cycle_graph(n).adj(Nat.0, Nat.1)
                    nat_range_set_contains_eq(n, Nat.0)
                    nat_range_set(n).contains(Nat.0)
                    nat_range_set_contains_eq(n, Nat.1)
                    nat_range_set(n).contains(Nat.1)
                    not_is_k_colorable_one_of_adj(cycle_graph(n), nat_range_set(n), Nat.0, Nat.1)
                    not is_k_colorable(cycle_graph(n), nat_range_set(n), Nat.1)
                    not is_k_colorable(cycle_graph(n), nat_range_set(n), j)
                }
                not is_k_colorable(cycle_graph(n), nat_range_set(n), j)
            }
            j < Nat.2 implies not is_k_colorable(cycle_graph(n), nat_range_set(n), j)
        }
        is_chromatic_number(cycle_graph(n), nat_range_set(n), Nat.2)
    }
}

// ============================================================================
// Odd cycles: a closed walk of odd length, so chi(C_n) >= 3 for odd n.
// ============================================================================

/// A proper two-coloring with values below two is a bipartition.
theorem proper_two_color_imp_is_bipartition[V](
    g: SimpleGraph[V], s: FiniteSet[V], color: V -> Nat
) {
    is_proper_coloring_on(g, s, color) and forall(v: V) {
        s.contains(v) implies color(v) < Nat.2
    } implies is_bipartition(g, s, part_of_two_color(color))
} by {
    if is_proper_coloring_on(g, s, color) and forall(v: V) {
        s.contains(v) implies color(v) < Nat.2
    } {
        is_bipartition_intro(g, s, part_of_two_color(color))
        forall(x: V, y: V) {
            if s.contains(x) and s.contains(y) and g.adj(x, y) {
                is_proper_coloring_on_apply(g, s, color, x, y)
                color(x) != color(y)
                color(x) < Nat.2
                color(y) < Nat.2
                two_color_ne_imp_one_eq_ne(color(x), color(y))
                (color(x) = Nat.1) != (color(y) = Nat.1)
                part_of_two_color_eq(color, x)
                part_of_two_color(color)(x) = (color(x) = Nat.1)
                part_of_two_color_eq(color, y)
                part_of_two_color(color)(y) = (color(y) = Nat.1)
                part_of_two_color(color)(x) != part_of_two_color(color)(y)
            }
            s.contains(x) and s.contains(y) and g.adj(x, y) implies part_of_two_color(color)(x) != part_of_two_color(color)(y)
        }
        is_bipartition(g, s, part_of_two_color(color))
    }
}

/// Successor, as a named function.
define nat_inc(v: Nat) -> Nat { v.suc }

/// Successor takes a natural to its successor.
theorem nat_inc_eq(v: Nat) {
    nat_inc(v) = v.suc
}

/// Successor is injective.
theorem nat_inc_injective {
    is_injective_fn(nat_inc)
} by {
    is_injective_fn(nat_inc) = forall(x: Nat, y: Nat) {
        nat_inc(x) = nat_inc(y) implies x = y
    }
    forall(x: Nat, y: Nat) {
        if nat_inc(x) = nat_inc(y) {
            nat_inc_eq(x)
            nat_inc(x) = x.suc
            nat_inc_eq(y)
            nat_inc(y) = y.suc
            x.suc = y.suc
            x.suc <= y.suc
            lte_cancel_suc(x, y)
            x <= y
            y.suc <= x.suc
            lte_cancel_suc(y, x)
            y <= x
            lte_antisymm(x, y)
            x = y
        }
        nat_inc(x) = nat_inc(y) implies x = y
    }
    is_injective_fn(nat_inc)
}

/// The predecessor of a positive natural closes the successor cycle.
theorem nat_sub_one_suc(n: Nat) {
    Nat.0 < n implies (n - Nat.1).suc = n
} by {
    if Nat.0 < n {
        if n = Nat.0 {
            Nat.0 < n
            Nat.0 < Nat.0
            lt_not_ref(Nat.0)
            false
        }
        n != Nat.0
        let (m: Nat) satisfy { m.suc = n }
        add_imp_sub(m, Nat.1, n)
        m + Nat.1 = n
        n - Nat.1 = m
        (n - Nat.1).suc = m.suc
        (n - Nat.1).suc = n
    }
}

/// Three vertices force at least two below the predecessor.
theorem nat_two_le_sub_one(n: Nat) {
    Nat.3 <= n implies Nat.2 <= n - Nat.1
} by {
    if Nat.3 <= n {
        Nat.0 < Nat.3
        lt_and_lte(Nat.0, Nat.3, n)
        Nat.0 < n
        if n = Nat.0 {
            Nat.0 < n
            Nat.0 < Nat.0
            lt_not_ref(Nat.0)
            false
        }
        n != Nat.0
        let (m: Nat) satisfy { m.suc = n }
        Nat.3 <= m.suc
        lte_cancel_suc(Nat.2, m)
        Nat.2 <= m
        add_imp_sub(m, Nat.1, n)
        m + Nat.1 = n
        n - Nat.1 = m
        Nat.2 <= n - Nat.1
    }
}

/// The walk from zero to `k` along the cycle steps through the successors.
theorem cycle_graph_walk_range(n: Nat, k: Nat) {
    k < n implies simple_graph_walk(cycle_graph(n), Nat.0, k, map(k.range, nat_inc))
} by {
    define p(kk: Nat) -> Bool {
        kk < n implies simple_graph_walk(cycle_graph(n), Nat.0, kk, map(kk.range, nat_inc))
    }

    if Nat.0 < n {
        Nat.0.range = List.nil[Nat]
        map(Nat.0.range, nat_inc) = map(List.nil[Nat], nat_inc)
        map(List.nil[Nat], nat_inc) = List.nil[Nat]
        map(Nat.0.range, nat_inc) = List.nil[Nat]
        simple_graph_walk_nil(cycle_graph(n), Nat.0, Nat.0)
        simple_graph_walk(cycle_graph(n), Nat.0, Nat.0, List.nil[Nat])
        simple_graph_walk(cycle_graph(n), Nat.0, Nat.0, map(Nat.0.range, nat_inc))
    }
    p(Nat.0)

    forall(kk: Nat) {
        if p(kk) {
            p(kk) = (kk < n implies simple_graph_walk(cycle_graph(n), Nat.0, kk, map(kk.range, nat_inc)))
            if kk.suc < n {
                lt_suc(kk)
                kk < kk.suc
                lt_trans(kk, kk.suc, n)
                kk < n
                simple_graph_walk(cycle_graph(n), Nat.0, kk, map(kk.range, nat_inc))
                cycle_graph_adj_suc(n, kk)
                cycle_graph(n).adj(kk, kk.suc)
                simple_graph_walk_append_adj(cycle_graph(n), Nat.0, kk, kk.suc, map(kk.range, nat_inc))
                simple_graph_walk(cycle_graph(n), Nat.0, kk.suc, map(kk.range, nat_inc).append(kk.suc))
                kk.suc.range = kk.range.append(kk)
                map_add(kk.range, List.singleton(kk), nat_inc)
                map(kk.range + List.singleton(kk), nat_inc) =
                    map(kk.range, nat_inc) + map(List.singleton(kk), nat_inc)
                map_singleton(nat_inc, kk)
                map(List.singleton(kk), nat_inc) = List.singleton(nat_inc(kk))
                map(kk.range, nat_inc) + List.singleton(nat_inc(kk)) =
                    map(kk.range, nat_inc).append(nat_inc(kk))
                nat_inc_eq(kk)
                nat_inc(kk) = kk.suc
                map(kk.range, nat_inc).append(kk.suc) = map(kk.suc.range, nat_inc)
                simple_graph_walk(cycle_graph(n), Nat.0, kk.suc, map(kk.suc.range, nat_inc))
            }
            p(kk.suc)
        }
    }
    forall(kk: Nat) { p(kk) implies p(kk.suc) }
    p(Nat.0) and forall(kk: Nat) { p(kk) implies p(kk.suc) }
    alt_induction(p)
    forall(kk: Nat) { p(kk) }
    p(k)
    p(k) = (k < n implies simple_graph_walk(cycle_graph(n), Nat.0, k, map(k.range, nat_inc)))
}

/// The cycle walk from zero back to zero through every vertex.
theorem cycle_graph_closed_walk_all(n: Nat) {
    Nat.3 <= n implies
        simple_graph_walk(cycle_graph(n), Nat.0, Nat.0,
            map((n - Nat.1).range, nat_inc).append(Nat.0))
} by {
    if Nat.3 <= n {
        Nat.0 < Nat.3
        lt_and_lte(Nat.0, Nat.3, n)
        Nat.0 < n
        sub_one_lt(n)
        n - Nat.1 < n
        cycle_graph_walk_range(n, n - Nat.1)
        simple_graph_walk(cycle_graph(n), Nat.0, n - Nat.1, map((n - Nat.1).range, nat_inc))
        nat_two_le_sub_one(n)
        Nat.2 <= n - Nat.1
        cycle_graph_adj_wrap(n - Nat.1)
        cycle_graph((n - Nat.1).suc).adj(n - Nat.1, Nat.0)
        nat_sub_one_suc(n)
        (n - Nat.1).suc = n
        cycle_graph(n).adj(n - Nat.1, Nat.0)
        simple_graph_walk_append_adj(cycle_graph(n), Nat.0, n - Nat.1, Nat.0,
            map((n - Nat.1).range, nat_inc))
        simple_graph_walk(cycle_graph(n), Nat.0, Nat.0,
            map((n - Nat.1).range, nat_inc).append(Nat.0))
    }
}

/// A walk whose every target lies in `s` stays inside `s`.
///
/// This is the membership-side of `walk_in_set`: instead of unfolding the recursive
/// definition of the predicate, one can establish pointwise membership of every target
/// of the walk and conclude `walk_in_set` directly.  Proved by structural induction on
/// the walk's list of targets.
theorem walk_in_set_of_all_targets_in[V](s: FiniteSet[V], steps: List[V]) {
    (forall(v: V) { steps.contains(v) implies s.contains(v) }) implies walk_in_set(s, steps)
} by {
    define pc(xs: List[V]) -> Bool {
        (forall(v: V) { xs.contains(v) implies s.contains(v) }) implies walk_in_set(s, xs)
    }

    walk_in_set_nil(s)
    pc(List.nil[V])

    forall(h: V, t: List[V]) {
        if pc(t) {
            if forall(v: V) { List.cons(h, t).contains(v) implies s.contains(v) } {
                cons_contains_head(h, t)
                List.cons(h, t).contains(h)
                s.contains(h)
                forall(v: V) {
                    if t.contains(v) {
                        cons_contains_of_tail_contains(h, t, v)
                        t.contains(v) implies List.cons(h, t).contains(v)
                        List.cons(h, t).contains(v)
                        s.contains(v)
                    }
                    t.contains(v) implies s.contains(v)
                }
                pc(t)
                walk_in_set(s, t)
                walk_in_set_cons(s, h, t)
                walk_in_set(s, List.cons(h, t)) = (s.contains(h) and walk_in_set(s, t))
                s.contains(h) and walk_in_set(s, t)
                walk_in_set(s, List.cons(h, t))
                pc(List.cons(h, t))
            }
            pc(List.cons(h, t))
        }
    }
    forall(h: V, t: List[V]) { pc(t) implies pc(List.cons(h, t)) }
    function[T0](x0: List[T0] -> Bool) {
        not forall(x1: T0, x2: List[T0]) { not x0(x2) or x0(List.cons(x1, x2)) } or
        not x0(List.nil[T0]) or forall(x3: List[T0]) { x0(x3) } = true
    }[V](pc)
    function[T0](x0: List[T0] -> Bool, x1: List[T0]) {
        not forall(x2: T0, x3: List[T0]) { not x0(x3) or x0(List.cons(x2, x3)) } or
        not x0(List.nil[T0]) or x0(x1)
    }[V](pc, steps)
    pc(steps)
}

/// The cycle walk stays inside the standard vertex set.
theorem cycle_graph_walk_all_in_set(n: Nat) {
    Nat.3 <= n implies
        walk_in_set(nat_range_set(n), map((n - Nat.1).range, nat_inc).append(Nat.0))
} by {
    if Nat.3 <= n {
        Nat.0 < Nat.3
        lt_and_lte(Nat.0, Nat.3, n)
        Nat.0 < n
        forall(v: Nat) {
            if map((n - Nat.1).range, nat_inc).append(Nat.0).contains(v) {
                add_contains_or(map((n - Nat.1).range, nat_inc), List.singleton(Nat.0), v)
                map((n - Nat.1).range, nat_inc).contains(v) or List.singleton(Nat.0).contains(v)
                if map((n - Nat.1).range, nat_inc).contains(v) {
                    map_contains((n - Nat.1).range, nat_inc, v)
                    let (x: Nat) satisfy {
                        (n - Nat.1).range.contains(x) and nat_inc(x) = v
                    }
                    nat_inc_eq(x)
                    nat_inc(x) = x.suc
                    v = x.suc
                    range_contains_iff_lt(n - Nat.1, x)
                    x < n - Nat.1
                    lt_imp_lte_suc(x, n - Nat.1)
                    x.suc <= n - Nat.1
                    v <= n - Nat.1
                    sub_one_lt(n)
                    n - Nat.1 < n
                    lte_and_lt(v, n - Nat.1, n)
                    v < n
                    nat_range_set_contains_eq(n, v)
                    nat_range_set(n).contains(v)
                }
                if List.singleton(Nat.0).contains(v) {
                    singleton_contains_imp_eq(Nat.0, v)
                    v = Nat.0
                    nat_range_set_contains_eq(n, Nat.0)
                    nat_range_set(n).contains(v)
                }
                nat_range_set(n).contains(v)
            }
            map((n - Nat.1).range, nat_inc).append(Nat.0).contains(v) implies nat_range_set(n).contains(v)
        }
        walk_in_set_of_all_targets_in(nat_range_set(n),
            map((n - Nat.1).range, nat_inc).append(Nat.0))
        walk_in_set(nat_range_set(n), map((n - Nat.1).range, nat_inc).append(Nat.0))
    }
}

/// The odd cycle is a closed walk of odd length.
theorem cycle_graph_odd_closed_walk(n: Nat) {
    Nat.3 <= n and nat_odd(n) implies
        simple_graph_closed_walk(cycle_graph(n), Nat.0,
            map((n - Nat.1).range, nat_inc).append(Nat.0))
} by {
    if Nat.3 <= n and nat_odd(n) {
        simple_graph_closed_walk_intro(cycle_graph(n), Nat.0,
            map((n - Nat.1).range, nat_inc).append(Nat.0))
        cycle_graph_closed_walk_all(n)
        simple_graph_walk(cycle_graph(n), Nat.0, Nat.0,
            map((n - Nat.1).range, nat_inc).append(Nat.0))
        map_length((n - Nat.1).range, nat_inc)
        map((n - Nat.1).range, nat_inc).length = (n - Nat.1).range.length
        length_range(n - Nat.1)
        (n - Nat.1).range.length = n - Nat.1
        map((n - Nat.1).range, nat_inc).length = n - Nat.1
        add_length(map((n - Nat.1).range, nat_inc), List.singleton(Nat.0))
        (map((n - Nat.1).range, nat_inc) + List.singleton(Nat.0)).length =
            map((n - Nat.1).range, nat_inc).length + List.singleton(Nat.0).length
        List.singleton(Nat.0).length = Nat.1
        map((n - Nat.1).range, nat_inc).append(Nat.0).length = n - Nat.1 + Nat.1
        Nat.0 < Nat.3
        lt_and_lte(Nat.0, Nat.3, n)
        Nat.0 < n
        nat_sub_one_suc(n)
        (n - Nat.1).suc = n
        n - Nat.1 + Nat.1 = n
        map((n - Nat.1).range, nat_inc).append(Nat.0).length = n
        Nat.0 < n
        map((n - Nat.1).range, nat_inc).append(Nat.0).length > Nat.0
        simple_graph_closed_walk(cycle_graph(n), Nat.0,
            map((n - Nat.1).range, nat_inc).append(Nat.0))
    }
}

/// An odd cycle is not two-colorable.
///
/// A two-coloring would be a bipartition, but a bipartition cannot contain a closed walk of
/// odd length: walking the cycle flips the side once per edge, so an odd closed walk would
/// leave a vertex on the opposite side of itself.
theorem cycle_graph_odd_not_k_colorable_two(n: Nat) {
    Nat.3 <= n and nat_odd(n) implies not is_k_colorable(cycle_graph(n), nat_range_set(n), Nat.2)
} by {
    if Nat.3 <= n and nat_odd(n) {
        if is_k_colorable(cycle_graph(n), nat_range_set(n), Nat.2) {
            is_k_colorable_witness(cycle_graph(n), nat_range_set(n), Nat.2)
            let (color: Nat -> Nat) satisfy {
                is_proper_coloring_on(cycle_graph(n), nat_range_set(n), color) and
                    forall(v: Nat) {
                        nat_range_set(n).contains(v) implies color(v) < Nat.2
                    }
            }
            proper_two_color_imp_is_bipartition(cycle_graph(n), nat_range_set(n), color)
            is_bipartition(cycle_graph(n), nat_range_set(n), part_of_two_color(color))
            cycle_graph_odd_closed_walk(n)
            simple_graph_closed_walk(cycle_graph(n), Nat.0,
                map((n - Nat.1).range, nat_inc).append(Nat.0))
            simple_graph_closed_walk_is_walk(cycle_graph(n), Nat.0,
                map((n - Nat.1).range, nat_inc).append(Nat.0))
            simple_graph_walk(cycle_graph(n), Nat.0, Nat.0,
                map((n - Nat.1).range, nat_inc).append(Nat.0))
            Nat.0 < Nat.3
            lt_and_lte(Nat.0, Nat.3, n)
            Nat.0 < n
            nat_range_set_contains_eq(n, Nat.0)
            nat_range_set(n).contains(Nat.0)
            cycle_graph_walk_all_in_set(n)
            walk_in_set(nat_range_set(n), map((n - Nat.1).range, nat_inc).append(Nat.0))
            bipartition_walk_color_flip(cycle_graph(n), nat_range_set(n),
                part_of_two_color(color), Nat.0, Nat.0,
                map((n - Nat.1).range, nat_inc).append(Nat.0))
            (part_of_two_color(color)(Nat.0) = part_of_two_color(color)(Nat.0)) =
                not nat_odd(map((n - Nat.1).range, nat_inc).append(Nat.0).length)
            map_length((n - Nat.1).range, nat_inc)
            map((n - Nat.1).range, nat_inc).length = (n - Nat.1).range.length
            length_range(n - Nat.1)
            (n - Nat.1).range.length = n - Nat.1
            map((n - Nat.1).range, nat_inc).length = n - Nat.1
            add_length(map((n - Nat.1).range, nat_inc), List.singleton(Nat.0))
            (map((n - Nat.1).range, nat_inc) + List.singleton(Nat.0)).length =
                map((n - Nat.1).range, nat_inc).length + List.singleton(Nat.0).length
            List.singleton(Nat.0).length = Nat.1
            map((n - Nat.1).range, nat_inc).append(Nat.0).length = n - Nat.1 + Nat.1
            nat_sub_one_suc(n)
            (n - Nat.1).suc = n
            n - Nat.1 + Nat.1 = n
            map((n - Nat.1).range, nat_inc).append(Nat.0).length = n
            not nat_odd(n)
            nat_odd(n)
            false
        }
        not is_k_colorable(cycle_graph(n), nat_range_set(n), Nat.2)
    }
}

// ============================================================================
// Odd cycles are three-colorable, so chi(C_n) = 3 for odd n.
// ============================================================================

/// A three-coloring of a cycle: parity colors, with the last vertex carrying color two.
define cycle_three_color(n: Nat) -> (Nat -> Nat) {
    function(v: Nat) {
        if v = n - Nat.1 {
            Nat.2
        } else {
            if nat_odd(v) {
                Nat.1
            } else {
                Nat.0
            }
        }
    }
}

/// The value of the three-coloring of a cycle.
theorem cycle_three_color_eq(n: Nat, v: Nat) {
    cycle_three_color(n)(v) = if v = n - Nat.1 {
        Nat.2
    } else {
        if nat_odd(v) {
            Nat.1
        } else {
            Nat.0
        }
    }
}

/// The closing vertex of the three-coloring carries color two.
theorem cycle_three_color_last(n: Nat) {
    cycle_three_color(n)(n - Nat.1) = Nat.2
} by {
    cycle_three_color_eq(n, n - Nat.1)
    cycle_three_color(n)(n - Nat.1) = if n - Nat.1 = n - Nat.1 {
        Nat.2
    } else {
        if nat_odd(n - Nat.1) {
            Nat.1
        } else {
            Nat.0
        }
    }
    cycle_three_color(n)(n - Nat.1) = Nat.2
}

/// An odd non-closing vertex of the three-coloring carries color one.
theorem cycle_three_color_parity(n: Nat, v: Nat) {
    v != n - Nat.1 and nat_odd(v) implies cycle_three_color(n)(v) = Nat.1
} by {
    if v != n - Nat.1 and nat_odd(v) {
        cycle_three_color_eq(n, v)
        cycle_three_color(n)(v) = if v = n - Nat.1 {
            Nat.2
        } else {
            if nat_odd(v) {
                Nat.1
            } else {
                Nat.0
            }
        }
        cycle_three_color(n)(v) = Nat.1
    }
}

/// An even non-closing vertex of the three-coloring carries color zero.
theorem cycle_three_color_even(n: Nat, v: Nat) {
    v != n - Nat.1 and not nat_odd(v) implies cycle_three_color(n)(v) = Nat.0
} by {
    if v != n - Nat.1 and not nat_odd(v) {
        cycle_three_color_eq(n, v)
        cycle_three_color(n)(v) = if v = n - Nat.1 {
            Nat.2
        } else {
            if nat_odd(v) {
                Nat.1
            } else {
                Nat.0
            }
        }
        cycle_three_color(n)(v) = Nat.0
    }
}

/// Successor equality cancels.
theorem nat_suc_eq_cancel(a: Nat, b: Nat) {
    a.suc = b.suc implies a = b
} by {
    if a.suc = b.suc {
        a.suc <= b.suc
        lte_cancel_suc(a, b)
        a <= b
        b.suc <= a.suc
        lte_cancel_suc(b, a)
        b <= a
        lte_antisymm(a, b)
        a = b
    }
}

/// The last vertex of a cycle is not the first.
theorem nat_sub_one_ne_zero(n: Nat) {
    Nat.3 <= n implies n - Nat.1 != Nat.0
} by {
    if Nat.3 <= n {
        nat_two_le_sub_one(n)
        Nat.2 <= n - Nat.1
        if n - Nat.1 = Nat.0 {
            Nat.2 <= Nat.0
            Nat.0 < Nat.2
            lte_and_lt(Nat.2, Nat.0, Nat.2)
            Nat.2 < Nat.2
            lt_not_ref(Nat.2)
            false
        }
        n - Nat.1 != Nat.0
    }
}

/// The three-coloring of a cycle separates the endpoints of a forward edge.
theorem cycle_graph_three_suc_color_ne(n: Nat, x: Nat, y: Nat) {
    Nat.3 <= n and x < n and y < n and x.suc.mod(n) = y
        implies cycle_three_color(n)(x) != cycle_three_color(n)(y)
} by {
    if Nat.3 <= n and x < n and y < n and x.suc.mod(n) = y {
        if x.suc < n {
            small_mod(x.suc, n)
            x.suc.mod(n) = x.suc
            y = x.suc
            if x.suc = n - Nat.1 {
                if x = n - Nat.1 {
                    x.suc = n
                    x.suc = n - Nat.1
                    n = n - Nat.1
                    sub_one_lt(n)
                    n - Nat.1 < n
                    n < n
                    lt_not_ref(n)
                    false
                }
                x != n - Nat.1
                cycle_three_color_last(n)
                cycle_three_color(n)(n - Nat.1) = Nat.2
                cycle_three_color(n)(y) = Nat.2
                if nat_odd(x) {
                    cycle_three_color_parity(n, x)
                    cycle_three_color(n)(x) = Nat.1
                    Nat.1 != Nat.2
                    cycle_three_color(n)(x) != Nat.2
                    cycle_three_color(n)(x) != cycle_three_color(n)(y)
                }
                if not nat_odd(x) {
                    cycle_three_color_even(n, x)
                    cycle_three_color(n)(x) = Nat.0
                    Nat.0 != Nat.2
                    cycle_three_color(n)(x) != Nat.2
                    cycle_three_color(n)(x) != cycle_three_color(n)(y)
                }
                cycle_three_color(n)(x) != cycle_three_color(n)(y)
            }
            if x.suc != n - Nat.1 {
                if x = n - Nat.1 {
                    Nat.0 < Nat.3
                    lt_and_lte(Nat.0, Nat.3, n)
                    Nat.0 < n
                    nat_sub_one_suc(n)
                    (n - Nat.1).suc = n
                    x.suc = (n - Nat.1).suc
                    x.suc = n
                    x.suc < n
                    n < n
                    lt_not_ref(n)
                    false
                }
                x != n - Nat.1
                y != n - Nat.1
                if nat_odd(x) {
                    cycle_three_color_parity(n, x)
                    cycle_three_color(n)(x) = Nat.1
                    nat_odd_suc(x)
                    nat_odd(x.suc) = not nat_odd(x)
                    nat_odd(y) = not nat_odd(x)
                    if nat_odd(y) {
                        nat_odd(y) = not nat_odd(x)
                        not nat_odd(x)
                        nat_odd(x)
                        false
                    }
                    not nat_odd(y)
                    cycle_three_color_even(n, y)
                    cycle_three_color(n)(y) = Nat.0
                    Nat.1 != Nat.0
                    cycle_three_color(n)(x) != cycle_three_color(n)(y)
                }
                if not nat_odd(x) {
                    cycle_three_color_even(n, x)
                    cycle_three_color(n)(x) = Nat.0
                    nat_odd_suc(x)
                    nat_odd(x.suc) = not nat_odd(x)
                    nat_odd(y) = not nat_odd(x)
                    if not nat_odd(y) {
                        nat_odd(y) = not nat_odd(x)
                        not nat_odd(y)
                        nat_odd(x)
                        not nat_odd(x)
                        false
                    }
                    nat_odd(y)
                    cycle_three_color_parity(n, y)
                    cycle_three_color(n)(y) = Nat.1
                    Nat.1 != Nat.0
                    cycle_three_color(n)(x) != cycle_three_color(n)(y)
                }
                cycle_three_color(n)(x) != cycle_three_color(n)(y)
            }
            cycle_three_color(n)(x) != cycle_three_color(n)(y)
        }
        if not (x.suc < n) {
            lt_imp_lte_suc(x, n)
            x.suc <= n
            lt_or_lte(x.suc, n)
            x.suc < n or n <= x.suc
            n <= x.suc
            lte_antisymm(x.suc, n)
            x.suc = n
            divides_self(n)
            div_imp_mod(n, n)
            n.mod(n) = Nat.0
            x.suc.mod(n) = n.mod(n)
            x.suc.mod(n) = Nat.0
            y = Nat.0
            nat_sub_one_suc(n)
            (n - Nat.1).suc = n
            x.suc = (n - Nat.1).suc
            nat_suc_eq_cancel(x, n - Nat.1)
            x = n - Nat.1
            cycle_three_color_last(n)
            cycle_three_color(n)(n - Nat.1) = Nat.2
            cycle_three_color(n)(x) = Nat.2
            nat_sub_one_ne_zero(n)
            n - Nat.1 != Nat.0
            if n - Nat.1 = Nat.0 {
                n - Nat.1 != Nat.0
                false
            }
            n - Nat.1 != Nat.0
            nat_odd_zero
            nat_odd(Nat.0) = false
            if nat_odd(Nat.0) {
                false
            }
            not nat_odd(Nat.0)
            cycle_three_color_even(n, Nat.0)
            cycle_three_color(n)(Nat.0) = Nat.0
            cycle_three_color(n)(y) = Nat.0
            Nat.2 != Nat.0
            cycle_three_color(n)(x) != cycle_three_color(n)(y)
        }
        cycle_three_color(n)(x) != cycle_three_color(n)(y)
    }
}

/// The three-coloring of a cycle separates the endpoints of every edge.
theorem cycle_graph_three_adj_color_ne(n: Nat, x: Nat, y: Nat) {
    Nat.3 <= n and x < n and y < n and cycle_graph(n).adj(x, y)
        implies cycle_three_color(n)(x) != cycle_three_color(n)(y)
} by {
    if Nat.3 <= n and x < n and y < n and cycle_graph(n).adj(x, y) {
        cycle_graph_adj_iff(n, x, y)
        x.suc.mod(n) = y or y.suc.mod(n) = x
        if x.suc.mod(n) = y {
            cycle_graph_three_suc_color_ne(n, x, y)
            cycle_three_color(n)(x) != cycle_three_color(n)(y)
        }
        if y.suc.mod(n) = x {
            cycle_graph_three_suc_color_ne(n, y, x)
            cycle_three_color(n)(y) != cycle_three_color(n)(x)
            cycle_three_color(n)(x) != cycle_three_color(n)(y)
        }
        cycle_three_color(n)(x) != cycle_three_color(n)(y)
    }
}

/// The three-coloring of a cycle uses fewer than three colors.
theorem cycle_three_color_lt_three(n: Nat, v: Nat) {
    Nat.3 <= n implies cycle_three_color(n)(v) < Nat.3
} by {
    if Nat.3 <= n {
        if v = n - Nat.1 {
            cycle_three_color_eq(n, v)
            cycle_three_color(n)(v) = if v = n - Nat.1 {
                Nat.2
            } else {
                if nat_odd(v) {
                    Nat.1
                } else {
                    Nat.0
                }
            }
            cycle_three_color(n)(v) = Nat.2
            Nat.2 < Nat.3
            cycle_three_color(n)(v) < Nat.3
        }
        if v != n - Nat.1 {
            if nat_odd(v) {
                cycle_three_color_eq(n, v)
                cycle_three_color(n)(v) = if v = n - Nat.1 {
                    Nat.2
                } else {
                    if nat_odd(v) {
                        Nat.1
                    } else {
                        Nat.0
                    }
                }
                cycle_three_color(n)(v) = Nat.1
                Nat.1 < Nat.2
                Nat.2 < Nat.3
                lt_trans(Nat.1, Nat.2, Nat.3)
                Nat.1 < Nat.3
                cycle_three_color(n)(v) < Nat.3
            }
            if not nat_odd(v) {
                cycle_three_color_eq(n, v)
                cycle_three_color(n)(v) = if v = n - Nat.1 {
                    Nat.2
                } else {
                    if nat_odd(v) {
                        Nat.1
                    } else {
                        Nat.0
                    }
                }
                cycle_three_color(n)(v) = Nat.0
                Nat.0 < Nat.3
                cycle_three_color(n)(v) < Nat.3
            }
            cycle_three_color(n)(v) < Nat.3
        }
        cycle_three_color(n)(v) < Nat.3
    }
}

/// A cycle on at least three vertices is three-colorable.
theorem cycle_graph_odd_is_k_colorable_three(n: Nat) {
    Nat.3 <= n implies is_k_colorable(cycle_graph(n), nat_range_set(n), Nat.3)
} by {
    if Nat.3 <= n {
        is_k_colorable_intro(cycle_graph(n), nat_range_set(n), Nat.3, cycle_three_color(n))
        is_proper_coloring_on_intro(cycle_graph(n), nat_range_set(n), cycle_three_color(n))
        forall(x: Nat, y: Nat) {
            if nat_range_set(n).contains(x) and nat_range_set(n).contains(y) and
                cycle_graph(n).adj(x, y) {
                nat_range_set_contains_eq(n, x)
                x < n
                nat_range_set_contains_eq(n, y)
                y < n
                cycle_graph_three_adj_color_ne(n, x, y)
                cycle_three_color(n)(x) != cycle_three_color(n)(y)
            }
            nat_range_set(n).contains(x) and nat_range_set(n).contains(y) and
                cycle_graph(n).adj(x, y) implies cycle_three_color(n)(x) != cycle_three_color(n)(y)
        }
        forall(v: Nat) {
            if nat_range_set(n).contains(v) {
                cycle_three_color_lt_three(n, v)
                cycle_three_color(n)(v) < Nat.3
            }
            nat_range_set(n).contains(v) implies cycle_three_color(n)(v) < Nat.3
        }
        is_proper_coloring_on(cycle_graph(n), nat_range_set(n), cycle_three_color(n))
        forall(v: Nat) { nat_range_set(n).contains(v) implies cycle_three_color(n)(v) < Nat.3 }
        is_k_colorable(cycle_graph(n), nat_range_set(n), Nat.3)
    }
}

/// A natural below three is zero, one, or two.
theorem nat_lt_three_cases(n: Nat) {
    n < Nat.3 implies n = Nat.0 or n = Nat.1 or n = Nat.2
} by {
    if n < Nat.3 {
        lt_suc_right(n, Nat.2)
        n = Nat.2 or n < Nat.2
        if n = Nat.2 {
            n = Nat.0 or n = Nat.1 or n = Nat.2
        }
        if n < Nat.2 {
            nat_lt_two_cases(n)
            n = Nat.0 or n = Nat.1
            if n = Nat.0 {
                n = Nat.0 or n = Nat.1 or n = Nat.2
            }
            if n = Nat.1 {
                n = Nat.0 or n = Nat.1 or n = Nat.2
            }
            n = Nat.0 or n = Nat.1 or n = Nat.2
        }
        n = Nat.0 or n = Nat.1 or n = Nat.2
    }
}

/// The chromatic number of an odd cycle is three.
theorem cycle_graph_odd_chromatic_number(n: Nat) {
    Nat.3 <= n and nat_odd(n) implies
        is_chromatic_number(cycle_graph(n), nat_range_set(n), Nat.3)
} by {
    if Nat.3 <= n and nat_odd(n) {
        is_chromatic_number_intro(cycle_graph(n), nat_range_set(n), Nat.3)
        cycle_graph_odd_is_k_colorable_three(n)
        is_k_colorable(cycle_graph(n), nat_range_set(n), Nat.3)
        forall(j: Nat) {
            if j < Nat.3 {
                nat_lt_three_cases(j)
                j = Nat.0 or j = Nat.1 or j = Nat.2
                if j = Nat.0 {
                    Nat.0 < Nat.3
                    lt_and_lte(Nat.0, Nat.3, n)
                    Nat.0 < n
                    nat_range_set_contains_eq(n, Nat.0)
                    nat_range_set(n).contains(Nat.0)
                    not_is_k_colorable_zero(cycle_graph(n), nat_range_set(n), Nat.0)
                    not is_k_colorable(cycle_graph(n), nat_range_set(n), Nat.0)
                    not is_k_colorable(cycle_graph(n), nat_range_set(n), j)
                }
                if j = Nat.1 {
                    Nat.1 < Nat.3
                    lt_and_lte(Nat.1, Nat.3, n)
                    Nat.1 < n
                    lt_suc(Nat.0)
                    Nat.0 < Nat.1
                    lt_trans(Nat.0, Nat.1, n)
                    Nat.0 < n
                    cycle_graph_adj_suc(n, Nat.0)
                    cycle_graph(n).adj(Nat.0, Nat.0.suc)
                    cycle_graph(n).adj(Nat.0, Nat.1)
                    nat_range_set_contains_eq(n, Nat.0)
                    nat_range_set(n).contains(Nat.0)
                    nat_range_set_contains_eq(n, Nat.1)
                    nat_range_set(n).contains(Nat.1)
                    not_is_k_colorable_one_of_adj(cycle_graph(n), nat_range_set(n), Nat.0, Nat.1)
                    not is_k_colorable(cycle_graph(n), nat_range_set(n), Nat.1)
                    not is_k_colorable(cycle_graph(n), nat_range_set(n), j)
                }
                if j = Nat.2 {
                    cycle_graph_odd_not_k_colorable_two(n)
                    not is_k_colorable(cycle_graph(n), nat_range_set(n), Nat.2)
                    not is_k_colorable(cycle_graph(n), nat_range_set(n), j)
                }
                not is_k_colorable(cycle_graph(n), nat_range_set(n), j)
            }
            j < Nat.3 implies not is_k_colorable(cycle_graph(n), nat_range_set(n), j)
        }
        is_chromatic_number(cycle_graph(n), nat_range_set(n), Nat.3)
    }
}

// ============================================================================
// The chromatic number as a total function: every finite vertex set admits a
// coloring with one color per vertex, so the minimum exists.
// ============================================================================
// The chromatic number as a total function: every finite vertex set admits a
// coloring with one color per vertex, so the minimum exists.
// ============================================================================

/// Distinct members of a unique list are found at distinct indices.
theorem find_first_idx_injective_of_unique[T](list: List[T], x: T, y: T) {
    list.is_unique and list.contains(x) and list.contains(y) and
        list.find_first_idx(x) = list.find_first_idx(y) implies x = y
} by {
    define p(xs: List[T]) -> Bool {
        forall(a: T, b: T) {
            xs.is_unique and xs.contains(a) and xs.contains(b) and
                xs.find_first_idx(a) = xs.find_first_idx(b) implies a = b
        }
    }

    forall(a: T, b: T) {
        if List.nil[T].is_unique and List.nil[T].contains(a) and List.nil[T].contains(b) and
            List.nil[T].find_first_idx(a) = List.nil[T].find_first_idx(b) {
            not List.nil[T].contains(a)
            if List.nil[T].contains(a) {
                false
            }
            false
        }
    }
    p(List.nil[T])

    forall(head: T, tail: List[T]) {
        if p(tail) {
            p(tail) = forall(a: T, b: T) {
                tail.is_unique and tail.contains(a) and tail.contains(b) and
                    tail.find_first_idx(a) = tail.find_first_idx(b) implies a = b
            }
            forall(a: T, b: T) {
                if List.cons(head, tail).is_unique and List.cons(head, tail).contains(a) and
                    List.cons(head, tail).contains(b) and
                    List.cons(head, tail).find_first_idx(a) = List.cons(head, tail).find_first_idx(b) {
                    unique_implies_tail_unique(head, tail)
                    tail.is_unique
                    List.cons(head, tail).find_first_idx(a) =
                        if head = a { Nat.0 } else { Nat.1 + tail.find_first_idx(a) }
                    List.cons(head, tail).find_first_idx(b) =
                        if head = b { Nat.0 } else { Nat.1 + tail.find_first_idx(b) }
                    if a = head {
                        List.cons(head, tail).find_first_idx(a) = Nat.0
                        List.cons(head, tail).find_first_idx(b) = Nat.0
                        if b = head {
                            b = a
                        }
                        if b != head {
                            List.cons(head, tail).find_first_idx(b) =
                                Nat.1 + tail.find_first_idx(b)
                            Nat.0 = Nat.1 + tail.find_first_idx(b)
                            false
                        }
                        a = b
                    }
                    if a != head {
                        if b = head {
                            List.cons(head, tail).find_first_idx(a) =
                                Nat.1 + tail.find_first_idx(a)
                            List.cons(head, tail).find_first_idx(b) = Nat.0
                            Nat.1 + tail.find_first_idx(a) = Nat.0
                            false
                        }
                        if b != head {
                            List.cons(head, tail).find_first_idx(a) =
                                Nat.1 + tail.find_first_idx(a)
                            List.cons(head, tail).find_first_idx(b) =
                                Nat.1 + tail.find_first_idx(b)
                            Nat.1 + tail.find_first_idx(a) = Nat.1 + tail.find_first_idx(b)
                            (Nat.1 + tail.find_first_idx(a)).suc =
                                (Nat.1 + tail.find_first_idx(b)).suc
                            tail.find_first_idx(a) = tail.find_first_idx(b)
                            cons_contains_eq(head, tail, a)
                            List.cons(head, tail).contains(a) = (a = head or tail.contains(a))
                            tail.contains(a)
                            cons_contains_eq(head, tail, b)
                            List.cons(head, tail).contains(b) = (b = head or tail.contains(b))
                            tail.contains(b)
                            tail.is_unique and tail.contains(a) and tail.contains(b) and tail.find_first_idx(a) = tail.find_first_idx(b) implies a = b
                            a = b
                        }
                        a = b
                    }
                    a = b
                }
                List.cons(head, tail).is_unique and List.cons(head, tail).contains(a) and
                    List.cons(head, tail).contains(b) and List.cons(head, tail).find_first_idx(a) = List.cons(head, tail).find_first_idx(b) implies a = b
            }
            p(List.cons(head, tail))
        }
    }
    forall(head: T, tail: List[T]) { p(tail) implies p(List.cons(head, tail)) }
    function[T0](x0: List[T0] -> Bool) {
        not forall(x1: T0, x2: List[T0]) { not x0(x2) or x0(List.cons(x1, x2)) } or
        not x0(List.nil[T0]) or forall(x3: List[T0]) { x0(x3) } = true
    }[T](p)
    function[T0](x0: List[T0] -> Bool, x1: List[T0]) {
        not forall(x2: T0, x3: List[T0]) { not x0(x3) or x0(List.cons(x2, x3)) } or
        not x0(List.nil[T0]) or x0(x1)
    }[T](p, list)
    p(list)
    p(list) = forall(a: T, b: T) {
        list.is_unique and list.contains(a) and list.contains(b) and
            list.find_first_idx(a) = list.find_first_idx(b) implies a = b
    }
    if list.is_unique and list.contains(x) and list.contains(y) and
        list.find_first_idx(x) = list.find_first_idx(y) {
        list.is_unique and list.contains(x) and list.contains(y) and
            list.find_first_idx(x) = list.find_first_idx(y) implies x = y
        x = y
    }
}

/// A contained item is found at an index below the length of the list.
theorem find_first_idx_lt_of_contains[T](list: List[T], item: T) {
    list.contains(item) implies list.find_first_idx(item) < list.length
} by {
    define p(xs: List[T]) -> Bool {
        xs.contains(item) implies xs.find_first_idx(item) < xs.length
    }

    if List.nil[T].contains(item) {
        not List.nil[T].contains(item)
        if List.nil[T].contains(item) {
            false
        }
        false
    }
    p(List.nil[T])

    forall(head: T, tail: List[T]) {
        if p(tail) {
            p(tail) = (tail.contains(item) implies tail.find_first_idx(item) < tail.length)
            if List.cons(head, tail).contains(item) {
                List.cons(head, tail).find_first_idx(item) =
                    if head = item { Nat.0 } else { Nat.1 + tail.find_first_idx(item) }
                List.cons(head, tail).length = tail.length + Nat.1
                if item = head {
                    List.cons(head, tail).find_first_idx(item) = Nat.0
                    Nat.0 < tail.length + Nat.1
                    Nat.0 < List.cons(head, tail).length
                    List.cons(head, tail).find_first_idx(item) < List.cons(head, tail).length
                }
                if item != head {
                    cons_contains_eq(head, tail, item)
                    List.cons(head, tail).contains(item) = (item = head or tail.contains(item))
                    tail.contains(item)
                    tail.find_first_idx(item) < tail.length
                    List.cons(head, tail).find_first_idx(item) =
                        Nat.1 + tail.find_first_idx(item)
                    Nat.1 + tail.find_first_idx(item) = tail.find_first_idx(item).suc
                    tail.find_first_idx(item).suc <= tail.length
                    Nat.1 + tail.find_first_idx(item) <= tail.length
                    lt_suc(tail.length)
                    tail.length < tail.length.suc
                    lte_and_lt(Nat.1 + tail.find_first_idx(item), tail.length, tail.length.suc)
                    Nat.1 + tail.find_first_idx(item) < tail.length.suc
                    List.cons(head, tail).find_first_idx(item) < tail.length.suc
                    tail.length.suc = List.cons(head, tail).length
                    List.cons(head, tail).find_first_idx(item) < List.cons(head, tail).length
                }
                List.cons(head, tail).find_first_idx(item) < List.cons(head, tail).length
            }
            p(List.cons(head, tail))
        }
    }
    forall(head: T, tail: List[T]) { p(tail) implies p(List.cons(head, tail)) }
    function[T0](x0: List[T0] -> Bool) {
        not forall(x1: T0, x2: List[T0]) { not x0(x2) or x0(List.cons(x1, x2)) } or
        not x0(List.nil[T0]) or forall(x3: List[T0]) { x0(x3) } = true
    }[T](p)
    function[T0](x0: List[T0] -> Bool, x1: List[T0]) {
        not forall(x2: T0, x3: List[T0]) { not x0(x3) or x0(List.cons(x2, x3)) } or
        not x0(List.nil[T0]) or x0(x1)
    }[T](p, list)
    p(list)
    p(list) = (list.contains(item) implies list.find_first_idx(item) < list.length)
}

/// The index of an item in a list, as a function of the item.
define list_index[V](items: List[V]) -> (V -> Nat) {
    function(v: V) {
        items.find_first_idx(v)
    }
}

/// The index of an item is its first occurrence.
theorem list_index_eq[V](items: List[V], v: V) {
    list_index(items)(v) = items.find_first_idx(v)
}

/// Every graph on a finite vertex set is colorable with one color per vertex: color each
/// vertex by its position in an enumeration of the vertex set.
theorem is_k_colorable_of_card[V](g: SimpleGraph[V], s: FiniteSet[V]) {
    is_k_colorable(g, s, fs_card(s))
} by {
    fs_card_cardinality_is(s)
    s.cardinality_is(fs_card(s))
    finite_set_cardinality_has_exact_unique_list(s, fs_card(s))
    exists(items: List[V]) {
        fs_from_list(items) = s and items.is_unique and items.length = fs_card(s)
    }
    let (items: List[V]) satisfy {
        fs_from_list(items) = s and items.is_unique and items.length = fs_card(s)
    }
    items.is_unique
    fs_from_list(items) = s
    items.length = fs_card(s)
    is_k_colorable_intro(g, s, fs_card(s), list_index(items))
    is_proper_coloring_on_intro(g, s, list_index(items))
    forall(x: V, y: V) {
        if s.contains(x) and s.contains(y) and g.adj(x, y) {
            simple_graph_adj_ne(g, x, y)
            x != y
            fs_from_list_contains_eq(items, x)
            fs_from_list(items).contains(x) = items.contains(x)
            s.contains(x)
            fs_from_list(items).contains(x)
            items.contains(x)
            fs_from_list_contains_eq(items, y)
            fs_from_list(items).contains(y) = items.contains(y)
            s.contains(y)
            fs_from_list(items).contains(y)
            items.contains(y)
            list_index_eq(items, x)
            list_index(items)(x) = items.find_first_idx(x)
            list_index_eq(items, y)
            list_index(items)(y) = items.find_first_idx(y)
            if list_index(items)(x) = list_index(items)(y) {
                items.find_first_idx(x) = items.find_first_idx(y)
                find_first_idx_injective_of_unique(items, x, y)
                x = y
                false
            }
            list_index(items)(x) != list_index(items)(y)
        }
        s.contains(x) and s.contains(y) and g.adj(x, y) implies list_index(items)(x) != list_index(items)(y)
    }
    forall(v: V) {
        if s.contains(v) {
            fs_from_list_contains_eq(items, v)
            fs_from_list(items).contains(v) = items.contains(v)
            s.contains(v)
            fs_from_list(items).contains(v)
            items.contains(v)
            find_first_idx_lt_of_contains(items, v)
            items.find_first_idx(v) < items.length
            list_index_eq(items, v)
            list_index(items)(v) = items.find_first_idx(v)
            list_index(items)(v) < items.length
            items.length = fs_card(s)
            list_index(items)(v) < fs_card(s)
        }
        s.contains(v) implies list_index(items)(v) < fs_card(s)
    }
    is_proper_coloring_on(g, s, list_index(items))
    forall(v: V) { s.contains(v) implies list_index(items)(v) < fs_card(s) }
    is_k_colorable(g, s, fs_card(s))
}

/// True of the numbers of colors that suffice to color `s`.
define chromatic_size_pred[V](g: SimpleGraph[V], s: FiniteSet[V]) -> (Nat -> Bool) {
    function(k: Nat) {
        is_k_colorable(g, s, k)
    }
}

/// A number of colors suffices exactly when it is a colorability.
theorem chromatic_size_pred_eq[V](g: SimpleGraph[V], s: FiniteSet[V], k: Nat) {
    chromatic_size_pred(g, s)(k) = is_k_colorable(g, s, k)
}

/// The chromatic number: the least number of colors of a proper coloring of `s`.
///
/// The least element exists because one color per vertex always suffices on a finite vertex
/// set, so the family of usable palette sizes is nonempty.
let chromatic_number[V](g: SimpleGraph[V], s: FiniteSet[V]) -> result: Nat satisfy {
    is_min(chromatic_size_pred(g, s), result)
} by {
    is_k_colorable_of_card(g, s)
    chromatic_size_pred_eq(g, s, fs_card(s))
    chromatic_size_pred(g, s)(fs_card(s))
    has_min(chromatic_size_pred(g, s), fs_card(s))
    exists(m: Nat) { is_min(chromatic_size_pred(g, s), m) }
}

/// The chromatic number is a chromatic number: it colors the graph and no smaller number does.
theorem chromatic_number_is_chromatic_number[V](g: SimpleGraph[V], s: FiniteSet[V]) {
    is_chromatic_number(g, s, chromatic_number(g, s))
} by {
    is_min(chromatic_size_pred(g, s), chromatic_number(g, s))
    is_min_apply(chromatic_size_pred(g, s), chromatic_number(g, s))
    chromatic_size_pred(g, s)(chromatic_number(g, s))
    chromatic_size_pred_eq(g, s, chromatic_number(g, s))
    is_k_colorable(g, s, chromatic_number(g, s))
    is_min_false_below(chromatic_size_pred(g, s), chromatic_number(g, s))
    false_below(chromatic_size_pred(g, s), chromatic_number(g, s))
    forall(j: Nat) {
        if j < chromatic_number(g, s) {
            false_below_apply(chromatic_size_pred(g, s), chromatic_number(g, s), j)
            not chromatic_size_pred(g, s)(j)
            chromatic_size_pred_eq(g, s, j)
            not is_k_colorable(g, s, j)
        }
        j < chromatic_number(g, s) implies not is_k_colorable(g, s, j)
    }
    is_chromatic_number_intro(g, s, chromatic_number(g, s))
    is_chromatic_number(g, s, chromatic_number(g, s))
}

/// The chromatic number of the complete graph on `n` vertices is `n`.
theorem complete_graph_chromatic_number_value(n: Nat) {
    chromatic_number(complete_graph[Nat], nat_range_set(n)) = n
} by {
    chromatic_number_is_chromatic_number(complete_graph[Nat], nat_range_set(n))
    is_chromatic_number(complete_graph[Nat], nat_range_set(n), chromatic_number(complete_graph[Nat], nat_range_set(n)))
    complete_graph_nat_range_chromatic_number(n)
    is_chromatic_number(complete_graph[Nat], nat_range_set(n), n)
    is_chromatic_number_unique(complete_graph[Nat], nat_range_set(n),
        chromatic_number(complete_graph[Nat], nat_range_set(n)), n)
    chromatic_number(complete_graph[Nat], nat_range_set(n)) = n
}


// ============================================================================
// The greedy bound and the four colour theorem.
//
// The following two classic statements are recorded here as statements only; their proofs
// are not yet formalized in this file.
//
// 1. The greedy bound. If every vertex of `s` has at most `delta` neighbours, then `delta + 1`
//    colours always suffice: colour the vertices of `s` one at a time, and when a vertex is
//    reached at most `delta` colours are already used on its neighbours, so at least one of the
//    `delta + 1` colours is free. In the language of this file the statement is:
//
//    // theorem greedy_coloring_bound[V](g: SimpleGraph[V], s: FiniteSet[V], delta: Nat) {
//    //     max_degree_at_most(g, s, delta) implies is_k_colorable(g, s, delta + Nat.1)
//    // }
//
//    The proof would formalize the greedy algorithm over the finite vertex set: the colouring
//    is built by induction on the elements of `s` (via the unique list representation used in
//    `is_k_colorable_of_card`), and the invariant is that each new vertex can be given the
//    least colour absent from its already-coloured neighbours. The colouring is total on `s`
//    because the vertex set is carried by the finite set `s`; the neighbourhood is the finite
//    set `neighborhood(g, s, v)` of `simple_graph_degree.ac`. This is left for future work.
//
// 2. The four colour theorem. Every planar graph is four-colourable. A planar graph is a graph
//    drawn in the plane so that edges meet only at vertices; the theorem says its chromatic
//    number is at most four. This library does not yet carry a notion of planarity, so the
//    statement cannot even be phrased here. The only proof known is the 1976 Appel--Haken
//    proof (reduced to a finite case check, later made machine-checked), far beyond the scope
//    of this file.
