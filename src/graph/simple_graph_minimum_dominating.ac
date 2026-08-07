from nat import Nat, lte_antisymm
from finite_set import FiniteSet
from data.finite.finite_set_card import fs_card
from graph.simple_graph import SimpleGraph
from graph.simple_graph_domination import is_dominating_set
from graph.simple_graph_domination_number import domination_number, dominating_size_pred,
    domination_number_attained, domination_number_is_least

numerals Nat

/// True if `d` is a dominating subset of `s`.
///
/// Named so that the minimum condition below is a two-part statement rather than a
/// three-part one, which keeps the conjunctions short enough to reason about.
define is_dominating_subset[V](
    g: SimpleGraph[V], s: FiniteSet[V], d: FiniteSet[V]
) -> Bool {
    d.subset_eq(s) and is_dominating_set(g, s, d)
}

/// A dominating subset is a subset that dominates.
theorem is_dominating_subset_apply[V](
    g: SimpleGraph[V], s: FiniteSet[V], d: FiniteSet[V]
) {
    is_dominating_subset(g, s, d) implies d.subset_eq(s) and is_dominating_set(g, s, d)
} by {
    if is_dominating_subset(g, s, d) {
        is_dominating_subset(g, s, d) = (d.subset_eq(s) and is_dominating_set(g, s, d))
        d.subset_eq(s) and is_dominating_set(g, s, d)
    }
}

/// A subset that dominates is a dominating subset.
theorem is_dominating_subset_intro[V](
    g: SimpleGraph[V], s: FiniteSet[V], d: FiniteSet[V]
) {
    d.subset_eq(s) and is_dominating_set(g, s, d) implies is_dominating_subset(g, s, d)
} by {
    if d.subset_eq(s) and is_dominating_set(g, s, d) {
        is_dominating_subset(g, s, d) = (d.subset_eq(s) and is_dominating_set(g, s, d))
        is_dominating_subset(g, s, d)
    }
}

/// No dominating subset is smaller than the domination number.
theorem dominating_subset_card_lower_bound[V](
    g: SimpleGraph[V], s: FiniteSet[V], d: FiniteSet[V]
) {
    is_dominating_subset(g, s, d) implies domination_number(g, s) <= fs_card(d)
} by {
    if is_dominating_subset(g, s, d) {
        is_dominating_subset_apply(g, s, d)
        d.subset_eq(s)
        is_dominating_set(g, s, d)
        domination_number_is_least(g, s, d)
        domination_number(g, s) <= fs_card(d)
    }
}

/// True if `d` is a dominating subset of `s` of the fewest possible size.
define is_minimum_dominating_set[V](
    g: SimpleGraph[V], s: FiniteSet[V], d: FiniteSet[V]
) -> Bool {
    is_dominating_subset(g, s, d) and fs_card(d) = domination_number(g, s)
}

/// A minimum dominating set is a dominating subset of the stated size.
theorem is_minimum_dominating_set_apply[V](
    g: SimpleGraph[V], s: FiniteSet[V], d: FiniteSet[V]
) {
    is_minimum_dominating_set(g, s, d)
        implies is_dominating_subset(g, s, d) and fs_card(d) = domination_number(g, s)
} by {
    if is_minimum_dominating_set(g, s, d) {
        is_minimum_dominating_set(g, s, d) =
            (is_dominating_subset(g, s, d) and fs_card(d) = domination_number(g, s))
        is_dominating_subset(g, s, d) and fs_card(d) = domination_number(g, s)
    }
}

/// The two conditions give a minimum dominating set.
theorem is_minimum_dominating_set_intro[V](
    g: SimpleGraph[V], s: FiniteSet[V], d: FiniteSet[V]
) {
    is_dominating_subset(g, s, d) and fs_card(d) = domination_number(g, s)
        implies is_minimum_dominating_set(g, s, d)
} by {
    if is_dominating_subset(g, s, d) and fs_card(d) = domination_number(g, s) {
        is_minimum_dominating_set(g, s, d) =
            (is_dominating_subset(g, s, d) and fs_card(d) = domination_number(g, s))
        is_minimum_dominating_set(g, s, d)
    }
}

/// A minimum dominating set exists.
///
/// The domination number is attained by construction, and the attaining set is exactly a
/// minimum dominating set.
theorem minimum_dominating_set_exists[V](g: SimpleGraph[V], s: FiniteSet[V]) {
    exists(d: FiniteSet[V]) { is_minimum_dominating_set(g, s, d) }
} by {
    domination_number_attained(g, s)
    dominating_size_pred(g, s)(domination_number(g, s))
    dominating_size_pred(g, s)(domination_number(g, s)) = exists(e: FiniteSet[V]) {
        e.subset_eq(s) and is_dominating_set(g, s, e) and fs_card(e) = domination_number(g, s)
    }
    exists(e: FiniteSet[V]) {
        e.subset_eq(s) and is_dominating_set(g, s, e) and fs_card(e) = domination_number(g, s)
    }
    let (d: FiniteSet[V]) satisfy {
        d.subset_eq(s) and is_dominating_set(g, s, d) and fs_card(d) = domination_number(g, s)
    }
    is_dominating_subset_intro(g, s, d)
    is_dominating_subset(g, s, d)
    is_minimum_dominating_set_intro(g, s, d)
    is_minimum_dominating_set(g, s, d)
    exists(e: FiniteSet[V]) { is_minimum_dominating_set(g, s, e) }
}

/// A dominating subset no larger than a minimum one is itself minimum.
///
/// The two bounds meet: no dominating subset falls below the domination number, and this one
/// does not rise above it.
theorem is_minimum_dominating_set_of_lte[V](
    g: SimpleGraph[V], s: FiniteSet[V], d: FiniteSet[V], e: FiniteSet[V]
) {
    is_minimum_dominating_set(g, s, e) and is_dominating_subset(g, s, d)
        and fs_card(d) <= fs_card(e)
        implies is_minimum_dominating_set(g, s, d)
} by {
    if is_minimum_dominating_set(g, s, e) and is_dominating_subset(g, s, d)
        and fs_card(d) <= fs_card(e) {
        is_minimum_dominating_set_apply(g, s, e)
        fs_card(e) = domination_number(g, s)
        fs_card(d) <= domination_number(g, s)
        dominating_subset_card_lower_bound(g, s, d)
        domination_number(g, s) <= fs_card(d)
        lte_antisymm(fs_card(d), domination_number(g, s))
        fs_card(d) = domination_number(g, s)
        is_minimum_dominating_set_intro(g, s, d)
        is_minimum_dominating_set(g, s, d)
    }
}

/// Any two minimum dominating sets have the same size.
theorem minimum_dominating_sets_same_size[V](
    g: SimpleGraph[V], s: FiniteSet[V], d: FiniteSet[V], e: FiniteSet[V]
) {
    is_minimum_dominating_set(g, s, d) and is_minimum_dominating_set(g, s, e)
        implies fs_card(d) = fs_card(e)
} by {
    if is_minimum_dominating_set(g, s, d) and is_minimum_dominating_set(g, s, e) {
        is_minimum_dominating_set_apply(g, s, d)
        fs_card(d) = domination_number(g, s)
        is_minimum_dominating_set_apply(g, s, e)
        fs_card(e) = domination_number(g, s)
        fs_card(d) = fs_card(e)
    }
}

/// True if only one subset attains the domination number.
///
/// Existence is automatic, so the content is the uniqueness clause alone. Graphs with a
/// unique minimum dominating set are the natural setting for questions about how far the
/// domination number can be forced to move.
define has_unique_minimum_dominating_set[V](g: SimpleGraph[V], s: FiniteSet[V]) -> Bool {
    forall(d: FiniteSet[V], e: FiniteSet[V]) {
        (is_minimum_dominating_set(g, s, d) and is_minimum_dominating_set(g, s, e)
            implies d = e)
    }
}

/// Two minimum dominating sets of a graph with a unique one coincide.
theorem has_unique_minimum_dominating_set_apply[V](
    g: SimpleGraph[V], s: FiniteSet[V], d: FiniteSet[V], e: FiniteSet[V]
) {
    has_unique_minimum_dominating_set(g, s) and is_minimum_dominating_set(g, s, d)
        and is_minimum_dominating_set(g, s, e)
        implies d = e
} by {
    if has_unique_minimum_dominating_set(g, s) and is_minimum_dominating_set(g, s, d)
        and is_minimum_dominating_set(g, s, e) {
        has_unique_minimum_dominating_set(g, s) = forall(a: FiniteSet[V], b: FiniteSet[V]) {
            (is_minimum_dominating_set(g, s, a) and is_minimum_dominating_set(g, s, b)
                implies a = b)
        }
        forall(a: FiniteSet[V], b: FiniteSet[V]) {
            (is_minimum_dominating_set(g, s, a) and is_minimum_dominating_set(g, s, b)
                implies a = b)
        }
        ((is_minimum_dominating_set(g, s, d) and is_minimum_dominating_set(g, s, e)
            implies d = e))
        d = e
    }
}

/// The pointwise uniqueness condition gives a unique minimum dominating set.
theorem has_unique_minimum_dominating_set_intro[V](g: SimpleGraph[V], s: FiniteSet[V]) {
    (forall(d: FiniteSet[V], e: FiniteSet[V]) {
        (is_minimum_dominating_set(g, s, d) and is_minimum_dominating_set(g, s, e)
            implies d = e)
    }) implies has_unique_minimum_dominating_set(g, s)
} by {
    if forall(d: FiniteSet[V], e: FiniteSet[V]) {
        (is_minimum_dominating_set(g, s, d) and is_minimum_dominating_set(g, s, e)
            implies d = e)
    } {
        has_unique_minimum_dominating_set(g, s) = forall(a: FiniteSet[V], b: FiniteSet[V]) {
            (is_minimum_dominating_set(g, s, a) and is_minimum_dominating_set(g, s, b)
                implies a = b)
        }
        has_unique_minimum_dominating_set(g, s)
    }
}

/// Under uniqueness, a dominating subset of minimum size is the minimum dominating set.
theorem unique_minimum_dominating_set_eq[V](
    g: SimpleGraph[V], s: FiniteSet[V], d: FiniteSet[V], e: FiniteSet[V]
) {
    has_unique_minimum_dominating_set(g, s) and is_minimum_dominating_set(g, s, e)
        and is_dominating_subset(g, s, d) and fs_card(d) <= fs_card(e)
        implies d = e
} by {
    if has_unique_minimum_dominating_set(g, s) and is_minimum_dominating_set(g, s, e)
        and is_dominating_subset(g, s, d) and fs_card(d) <= fs_card(e) {
        is_minimum_dominating_set_of_lte(g, s, d, e)
        is_minimum_dominating_set(g, s, d)
        has_unique_minimum_dominating_set_apply(g, s, d, e)
        d = e
    }
}
