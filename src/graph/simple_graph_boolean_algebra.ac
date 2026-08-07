from graph.simple_graph import SimpleGraph, simple_graph_ext, simple_graph_adj_ne,
    empty_graph, complete_graph, graph_complement, graph_union, graph_intersection,
    empty_graph_adj_false, complete_graph_adj_iff_ne, graph_complement_adj_eq,
    graph_union_adj_eq, graph_intersection_adj_eq

/// Union of simple graphs is associative.
theorem graph_union_assoc[V](g: SimpleGraph[V], h: SimpleGraph[V], k: SimpleGraph[V]) {
    graph_union(graph_union(g, h), k) = graph_union(g, graph_union(h, k))
} by {
    forall(x: V, y: V) {
        graph_union_adj_eq(graph_union(g, h), k, x, y)
        graph_union_adj_eq(g, h, x, y)
        graph_union_adj_eq(g, graph_union(h, k), x, y)
        graph_union_adj_eq(h, k, x, y)
        if graph_union(graph_union(g, h), k).adj(x, y) {
            if graph_union(g, h).adj(x, y) {
                if g.adj(x, y) {
                    graph_union(g, graph_union(h, k)).adj(x, y)
                } else {
                    h.adj(x, y)
                    graph_union(h, k).adj(x, y)
                    graph_union(g, graph_union(h, k)).adj(x, y)
                }
            } else {
                k.adj(x, y)
                graph_union(h, k).adj(x, y)
                graph_union(g, graph_union(h, k)).adj(x, y)
            }
        }
        if graph_union(g, graph_union(h, k)).adj(x, y) {
            if g.adj(x, y) {
                graph_union(g, h).adj(x, y)
                graph_union(graph_union(g, h), k).adj(x, y)
            } else {
                graph_union(h, k).adj(x, y)
                if h.adj(x, y) {
                    graph_union(g, h).adj(x, y)
                    graph_union(graph_union(g, h), k).adj(x, y)
                } else {
                    k.adj(x, y)
                    graph_union(graph_union(g, h), k).adj(x, y)
                }
            }
        }
        graph_union(graph_union(g, h), k).adj(x, y) =
            graph_union(g, graph_union(h, k)).adj(x, y)
    }
    simple_graph_ext(graph_union(graph_union(g, h), k), graph_union(g, graph_union(h, k)))
}

/// Intersection of simple graphs is associative.
theorem graph_intersection_assoc[V](g: SimpleGraph[V], h: SimpleGraph[V], k: SimpleGraph[V]) {
    graph_intersection(graph_intersection(g, h), k) = graph_intersection(g, graph_intersection(h, k))
} by {
    forall(x: V, y: V) {
        graph_intersection_adj_eq(graph_intersection(g, h), k, x, y)
        graph_intersection_adj_eq(g, h, x, y)
        graph_intersection_adj_eq(g, graph_intersection(h, k), x, y)
        graph_intersection_adj_eq(h, k, x, y)
        if graph_intersection(graph_intersection(g, h), k).adj(x, y) {
            graph_intersection(g, h).adj(x, y)
            k.adj(x, y)
            g.adj(x, y)
            h.adj(x, y)
            graph_intersection(h, k).adj(x, y)
            graph_intersection(g, graph_intersection(h, k)).adj(x, y)
        }
        if graph_intersection(g, graph_intersection(h, k)).adj(x, y) {
            g.adj(x, y)
            graph_intersection(h, k).adj(x, y)
            h.adj(x, y)
            k.adj(x, y)
            graph_intersection(g, h).adj(x, y)
            graph_intersection(graph_intersection(g, h), k).adj(x, y)
        }
        graph_intersection(graph_intersection(g, h), k).adj(x, y) =
            graph_intersection(g, graph_intersection(h, k)).adj(x, y)
    }
    simple_graph_ext(graph_intersection(graph_intersection(g, h), k), graph_intersection(g, graph_intersection(h, k)))
}

/// A graph union the intersection of itself with another graph is the original graph.
theorem graph_union_intersection_absorb[V](g: SimpleGraph[V], h: SimpleGraph[V]) {
    graph_union(g, graph_intersection(g, h)) = g
} by {
    forall(x: V, y: V) {
        graph_union_adj_eq(g, graph_intersection(g, h), x, y)
        graph_intersection_adj_eq(g, h, x, y)
        if graph_union(g, graph_intersection(g, h)).adj(x, y) {
            if g.adj(x, y) {
                g.adj(x, y)
            } else {
                graph_intersection(g, h).adj(x, y)
                g.adj(x, y)
            }
        }
        if g.adj(x, y) {
            graph_union(g, graph_intersection(g, h)).adj(x, y)
        }
        graph_union(g, graph_intersection(g, h)).adj(x, y) = g.adj(x, y)
    }
    simple_graph_ext(graph_union(g, graph_intersection(g, h)), g)
}

/// A graph intersect the union of itself with another graph is the original graph.
theorem graph_intersection_union_absorb[V](g: SimpleGraph[V], h: SimpleGraph[V]) {
    graph_intersection(g, graph_union(g, h)) = g
} by {
    forall(x: V, y: V) {
        graph_intersection_adj_eq(g, graph_union(g, h), x, y)
        graph_union_adj_eq(g, h, x, y)
        if graph_intersection(g, graph_union(g, h)).adj(x, y) {
            g.adj(x, y)
        }
        if g.adj(x, y) {
            graph_union(g, h).adj(x, y)
            graph_intersection(g, graph_union(g, h)).adj(x, y)
        }
        graph_intersection(g, graph_union(g, h)).adj(x, y) = g.adj(x, y)
    }
    simple_graph_ext(graph_intersection(g, graph_union(g, h)), g)
}

/// Graph union distributes over graph intersection.
theorem graph_union_intersection_distrib[V](
    g: SimpleGraph[V],
    h: SimpleGraph[V],
    k: SimpleGraph[V]
) {
    graph_union(g, graph_intersection(h, k)) =
        graph_intersection(graph_union(g, h), graph_union(g, k))
} by {
    forall(x: V, y: V) {
        graph_union_adj_eq(g, graph_intersection(h, k), x, y)
        graph_intersection_adj_eq(h, k, x, y)
        graph_intersection_adj_eq(graph_union(g, h), graph_union(g, k), x, y)
        graph_union_adj_eq(g, h, x, y)
        graph_union_adj_eq(g, k, x, y)
        if graph_union(g, graph_intersection(h, k)).adj(x, y) {
            if g.adj(x, y) {
                graph_union(g, h).adj(x, y)
                graph_union(g, k).adj(x, y)
                graph_intersection(graph_union(g, h), graph_union(g, k)).adj(x, y)
            } else {
                graph_intersection(h, k).adj(x, y)
                h.adj(x, y)
                k.adj(x, y)
                graph_union(g, h).adj(x, y)
                graph_union(g, k).adj(x, y)
                graph_intersection(graph_union(g, h), graph_union(g, k)).adj(x, y)
            }
        }
        if graph_intersection(graph_union(g, h), graph_union(g, k)).adj(x, y) {
            graph_union(g, h).adj(x, y)
            graph_union(g, k).adj(x, y)
            if g.adj(x, y) {
                graph_union(g, graph_intersection(h, k)).adj(x, y)
            } else {
                h.adj(x, y)
                k.adj(x, y)
                graph_intersection(h, k).adj(x, y)
                graph_union(g, graph_intersection(h, k)).adj(x, y)
            }
        }
        graph_union(g, graph_intersection(h, k)).adj(x, y) =
            graph_intersection(graph_union(g, h), graph_union(g, k)).adj(x, y)
    }
    simple_graph_ext(
        graph_union(g, graph_intersection(h, k)),
        graph_intersection(graph_union(g, h), graph_union(g, k))
    )
}

/// Graph intersection distributes over graph union.
theorem graph_intersection_union_distrib[V](
    g: SimpleGraph[V],
    h: SimpleGraph[V],
    k: SimpleGraph[V]
) {
    graph_intersection(g, graph_union(h, k)) =
        graph_union(graph_intersection(g, h), graph_intersection(g, k))
} by {
    forall(x: V, y: V) {
        graph_intersection_adj_eq(g, graph_union(h, k), x, y)
        graph_union_adj_eq(h, k, x, y)
        graph_union_adj_eq(graph_intersection(g, h), graph_intersection(g, k), x, y)
        graph_intersection_adj_eq(g, h, x, y)
        graph_intersection_adj_eq(g, k, x, y)
        if graph_intersection(g, graph_union(h, k)).adj(x, y) {
            g.adj(x, y)
            graph_union(h, k).adj(x, y)
            if h.adj(x, y) {
                graph_intersection(g, h).adj(x, y)
                graph_union(graph_intersection(g, h), graph_intersection(g, k)).adj(x, y)
            } else {
                k.adj(x, y)
                graph_intersection(g, k).adj(x, y)
                graph_union(graph_intersection(g, h), graph_intersection(g, k)).adj(x, y)
            }
        }
        if graph_union(graph_intersection(g, h), graph_intersection(g, k)).adj(x, y) {
            if graph_intersection(g, h).adj(x, y) {
                g.adj(x, y)
                h.adj(x, y)
                graph_union(h, k).adj(x, y)
                graph_intersection(g, graph_union(h, k)).adj(x, y)
            } else {
                graph_intersection(g, k).adj(x, y)
                g.adj(x, y)
                k.adj(x, y)
                graph_union(h, k).adj(x, y)
                graph_intersection(g, graph_union(h, k)).adj(x, y)
            }
        }
        graph_intersection(g, graph_union(h, k)).adj(x, y) =
            graph_union(graph_intersection(g, h), graph_intersection(g, k)).adj(x, y)
    }
    simple_graph_ext(
        graph_intersection(g, graph_union(h, k)),
        graph_union(graph_intersection(g, h), graph_intersection(g, k))
    )
}

/// A graph union its complement is the complete graph.
theorem graph_union_complement[V](g: SimpleGraph[V]) {
    graph_union(g, graph_complement(g)) = complete_graph[V]
} by {
    forall(x: V, y: V) {
        graph_union_adj_eq(g, graph_complement(g), x, y)
        graph_complement_adj_eq(g, x, y)
        complete_graph_adj_iff_ne[V](x, y)
        if graph_union(g, graph_complement(g)).adj(x, y) {
            if g.adj(x, y) {
                simple_graph_adj_ne(g, x, y)
                x != y
                complete_graph[V].adj(x, y)
            } else {
                graph_complement(g).adj(x, y)
                x != y
                complete_graph[V].adj(x, y)
            }
        }
        if complete_graph[V].adj(x, y) {
            x != y
            if g.adj(x, y) {
                graph_union(g, graph_complement(g)).adj(x, y)
            } else {
                not g.adj(x, y)
                graph_complement(g).adj(x, y)
                graph_union(g, graph_complement(g)).adj(x, y)
            }
        }
        graph_union(g, graph_complement(g)).adj(x, y) = complete_graph[V].adj(x, y)
    }
    simple_graph_ext(graph_union(g, graph_complement(g)), complete_graph[V])
}

/// A graph intersect its complement is the empty graph.
theorem graph_intersection_complement[V](g: SimpleGraph[V]) {
    graph_intersection(g, graph_complement(g)) = empty_graph[V]
} by {
    forall(x: V, y: V) {
        graph_intersection_adj_eq(g, graph_complement(g), x, y)
        graph_complement_adj_eq(g, x, y)
        empty_graph_adj_false[V](x, y)
        if graph_intersection(g, graph_complement(g)).adj(x, y) {
            g.adj(x, y)
            graph_complement(g).adj(x, y)
            not g.adj(x, y)
            false
        }
        if empty_graph[V].adj(x, y) {
            false
        }
        graph_intersection(g, graph_complement(g)).adj(x, y) = empty_graph[V].adj(x, y)
    }
    simple_graph_ext(graph_intersection(g, graph_complement(g)), empty_graph[V])
}

/// The complement of a union is the intersection of the complements.
theorem graph_complement_union[V](g: SimpleGraph[V], h: SimpleGraph[V]) {
    graph_complement(graph_union(g, h)) =
        graph_intersection(graph_complement(g), graph_complement(h))
} by {
    forall(x: V, y: V) {
        graph_complement_adj_eq(graph_union(g, h), x, y)
        graph_union_adj_eq(g, h, x, y)
        graph_intersection_adj_eq(graph_complement(g), graph_complement(h), x, y)
        graph_complement_adj_eq(g, x, y)
        graph_complement_adj_eq(h, x, y)
        if graph_complement(graph_union(g, h)).adj(x, y) {
            x != y
            not graph_union(g, h).adj(x, y)
            if g.adj(x, y) {
                graph_union(g, h).adj(x, y)
                false
            }
            not g.adj(x, y)
            if h.adj(x, y) {
                graph_union(g, h).adj(x, y)
                false
            }
            not h.adj(x, y)
            graph_complement(g).adj(x, y)
            graph_complement(h).adj(x, y)
            graph_intersection(graph_complement(g), graph_complement(h)).adj(x, y)
        }
        if graph_intersection(graph_complement(g), graph_complement(h)).adj(x, y) {
            graph_complement(g).adj(x, y)
            graph_complement(h).adj(x, y)
            x != y
            not g.adj(x, y)
            not h.adj(x, y)
            if graph_union(g, h).adj(x, y) {
                if g.adj(x, y) {
                    false
                } else {
                    h.adj(x, y)
                    false
                }
            }
            not graph_union(g, h).adj(x, y)
            graph_complement(graph_union(g, h)).adj(x, y)
        }
        graph_complement(graph_union(g, h)).adj(x, y) =
            graph_intersection(graph_complement(g), graph_complement(h)).adj(x, y)
    }
    simple_graph_ext(graph_complement(graph_union(g, h)), graph_intersection(graph_complement(g), graph_complement(h)))
}

/// The complement of an intersection is the union of the complements.
theorem graph_complement_intersection[V](g: SimpleGraph[V], h: SimpleGraph[V]) {
    graph_complement(graph_intersection(g, h)) =
        graph_union(graph_complement(g), graph_complement(h))
} by {
    forall(x: V, y: V) {
        graph_complement_adj_eq(graph_intersection(g, h), x, y)
        graph_intersection_adj_eq(g, h, x, y)
        graph_union_adj_eq(graph_complement(g), graph_complement(h), x, y)
        graph_complement_adj_eq(g, x, y)
        graph_complement_adj_eq(h, x, y)
        if graph_complement(graph_intersection(g, h)).adj(x, y) {
            x != y
            not graph_intersection(g, h).adj(x, y)
            if g.adj(x, y) {
                if h.adj(x, y) {
                    graph_intersection(g, h).adj(x, y)
                    false
                }
                not h.adj(x, y)
                graph_complement(h).adj(x, y)
                graph_union(graph_complement(g), graph_complement(h)).adj(x, y)
            } else {
                not g.adj(x, y)
                graph_complement(g).adj(x, y)
                graph_union(graph_complement(g), graph_complement(h)).adj(x, y)
            }
        }
        if graph_union(graph_complement(g), graph_complement(h)).adj(x, y) {
            if graph_complement(g).adj(x, y) {
                x != y
                not g.adj(x, y)
                if graph_intersection(g, h).adj(x, y) {
                    g.adj(x, y)
                    false
                }
                not graph_intersection(g, h).adj(x, y)
                graph_complement(graph_intersection(g, h)).adj(x, y)
            } else {
                graph_complement(h).adj(x, y)
                x != y
                not h.adj(x, y)
                if graph_intersection(g, h).adj(x, y) {
                    h.adj(x, y)
                    false
                }
                not graph_intersection(g, h).adj(x, y)
                graph_complement(graph_intersection(g, h)).adj(x, y)
            }
        }
        graph_complement(graph_intersection(g, h)).adj(x, y) =
            graph_union(graph_complement(g), graph_complement(h)).adj(x, y)
    }
    simple_graph_ext(graph_complement(graph_intersection(g, h)), graph_union(graph_complement(g), graph_complement(h)))
}
