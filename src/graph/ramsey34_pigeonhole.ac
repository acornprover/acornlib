from nat import Nat
from data.basic.logic import exists_intro
from graph.ramsey34_good import good_four, good_four_intro,
good_four_1234, good_four_1235, good_four_1236, good_four_1237, good_four_1238, good_four_1245, good_four_1246,
    good_four_1247, good_four_1248, good_four_1256, good_four_1257, good_four_1258, good_four_1267, good_four_1268,
    good_four_1278, good_four_1345, good_four_1346, good_four_1347, good_four_1348, good_four_1356, good_four_1357,
    good_four_1358, good_four_1367, good_four_1368, good_four_1378, good_four_1456, good_four_1457, good_four_1458,
    good_four_1467, good_four_1468, good_four_1478, good_four_1567, good_four_1568, good_four_1578, good_four_1678,
    good_four_2345, good_four_2346, good_four_2347, good_four_2348, good_four_2356, good_four_2357, good_four_2358,
    good_four_2367, good_four_2368, good_four_2378, good_four_2456, good_four_2457, good_four_2458, good_four_2467,
    good_four_2468, good_four_2478, good_four_2567, good_four_2568, good_four_2578, good_four_2678, good_four_3456,
    good_four_3457, good_four_3458, good_four_3467, good_four_3468, good_four_3478, good_four_3567, good_four_3568,
    good_four_3578, good_four_3678, good_four_4567, good_four_4568, good_four_4578, good_four_4678, good_four_5678

numerals Nat

/// True if four of the eight edges from vertex 0 to 1, ..., 8 share a color.
define four_edges_from_zero_share_color(color: (Nat, Nat) -> Bool) -> Bool {
    exists(w: Nat, x: Nat, y: Nat, z: Nat) {
        good_four(w, x, y, z)
        and color(Nat.0, w) = color(Nat.0, x) and color(Nat.0, x) = color(Nat.0, y)
        and color(Nat.0, y) = color(Nat.0, z)
    }
}

/// Four good vertices with equal edge colors to 0 witness the pigeonhole.
theorem four_edges_from_zero_share_color_intro(color: (Nat, Nat) -> Bool, w: Nat, x: Nat, y: Nat, z: Nat) {
    good_four(w, x, y, z)
        and color(Nat.0, w) = color(Nat.0, x) and color(Nat.0, x) = color(Nat.0, y)
        and color(Nat.0, y) = color(Nat.0, z)
        implies four_edges_from_zero_share_color(color)
} by {
    if good_four(w, x, y, z)
        and color(Nat.0, w) = color(Nat.0, x) and color(Nat.0, x) = color(Nat.0, y)
        and color(Nat.0, y) = color(Nat.0, z) {
        four_edges_from_zero_share_color(color) = exists(a: Nat, b: Nat, c: Nat, d: Nat) {
            good_four(a, b, c, d)
            and color(Nat.0, a) = color(Nat.0, b) and color(Nat.0, b) = color(Nat.0, c)
            and color(Nat.0, c) = color(Nat.0, d)
        }
        exists_intro(function(a: Nat) {
            exists(b: Nat, c: Nat, d: Nat) {
                good_four(a, b, c, d)
                and color(Nat.0, a) = color(Nat.0, b) and color(Nat.0, b) = color(Nat.0, c)
                and color(Nat.0, c) = color(Nat.0, d)
            }
        }, w)
        exists_intro(function(b: Nat) {
            exists(c: Nat, d: Nat) {
                good_four(w, b, c, d)
                and color(Nat.0, w) = color(Nat.0, b) and color(Nat.0, b) = color(Nat.0, c)
                and color(Nat.0, c) = color(Nat.0, d)
            }
        }, x)
        exists_intro(function(c: Nat) {
            exists(d: Nat) {
                good_four(w, x, c, d)
                and color(Nat.0, w) = color(Nat.0, x) and color(Nat.0, x) = color(Nat.0, c)
                and color(Nat.0, c) = color(Nat.0, d)
            }
        }, y)
        exists_intro(function(d: Nat) {
            good_four(w, x, y, d)
            and color(Nat.0, w) = color(Nat.0, x) and color(Nat.0, x) = color(Nat.0, y)
            and color(Nat.0, y) = color(Nat.0, d)
        }, z)
        exists(a: Nat, b: Nat, c: Nat, d: Nat) {
            good_four(a, b, c, d)
            and color(Nat.0, a) = color(Nat.0, b) and color(Nat.0, b) = color(Nat.0, c)
            and color(Nat.0, c) = color(Nat.0, d)
        }
        four_edges_from_zero_share_color(color)
    }
}

/// Among the eight edges from vertex 0 to 1, ..., 8, four share a color.
///
/// The eight edge colors are case-split directly; each branch names four edges of
/// a common color. Only two colors exist, so eight edges must contain four of one
/// color. Each branch uses the pre-proved vertex-quality facts for its four
/// vertices and applies the pigeonhole introduction lemma.
theorem four_edges_from_zero_share_color_pigeonhole(color: (Nat, Nat) -> Bool) {
    four_edges_from_zero_share_color(color)
} by {
        if color(Nat.0, Nat.1) {
        if color(Nat.0, Nat.2) {
        if color(Nat.0, Nat.3) {
        if color(Nat.0, Nat.4) {
        if color(Nat.0, Nat.5) {
        if color(Nat.0, Nat.6) {
        if color(Nat.0, Nat.7) {
        if color(Nat.0, Nat.8) {
        color(Nat.0, Nat.1) = color(Nat.0, Nat.2)
        color(Nat.0, Nat.2) = color(Nat.0, Nat.3)
        color(Nat.0, Nat.3) = color(Nat.0, Nat.4)
        good_four_1234
        good_four(Nat.1, Nat.2, Nat.3, Nat.4)
        four_edges_from_zero_share_color_intro(color, Nat.1, Nat.2, Nat.3, Nat.4)
        four_edges_from_zero_share_color(color)
        } else {
        color(Nat.0, Nat.1) = color(Nat.0, Nat.2)
        color(Nat.0, Nat.2) = color(Nat.0, Nat.3)
        color(Nat.0, Nat.3) = color(Nat.0, Nat.4)
        good_four_1234
        good_four(Nat.1, Nat.2, Nat.3, Nat.4)
        four_edges_from_zero_share_color_intro(color, Nat.1, Nat.2, Nat.3, Nat.4)
        four_edges_from_zero_share_color(color)
        }
        } else {
        if color(Nat.0, Nat.8) {
        color(Nat.0, Nat.1) = color(Nat.0, Nat.2)
        color(Nat.0, Nat.2) = color(Nat.0, Nat.3)
        color(Nat.0, Nat.3) = color(Nat.0, Nat.4)
        good_four_1234
        good_four(Nat.1, Nat.2, Nat.3, Nat.4)
        four_edges_from_zero_share_color_intro(color, Nat.1, Nat.2, Nat.3, Nat.4)
        four_edges_from_zero_share_color(color)
        } else {
        color(Nat.0, Nat.1) = color(Nat.0, Nat.2)
        color(Nat.0, Nat.2) = color(Nat.0, Nat.3)
        color(Nat.0, Nat.3) = color(Nat.0, Nat.4)
        good_four_1234
        good_four(Nat.1, Nat.2, Nat.3, Nat.4)
        four_edges_from_zero_share_color_intro(color, Nat.1, Nat.2, Nat.3, Nat.4)
        four_edges_from_zero_share_color(color)
        }
        }
        } else {
        if color(Nat.0, Nat.7) {
        if color(Nat.0, Nat.8) {
        color(Nat.0, Nat.1) = color(Nat.0, Nat.2)
        color(Nat.0, Nat.2) = color(Nat.0, Nat.3)
        color(Nat.0, Nat.3) = color(Nat.0, Nat.4)
        good_four_1234
        good_four(Nat.1, Nat.2, Nat.3, Nat.4)
        four_edges_from_zero_share_color_intro(color, Nat.1, Nat.2, Nat.3, Nat.4)
        four_edges_from_zero_share_color(color)
        } else {
        color(Nat.0, Nat.1) = color(Nat.0, Nat.2)
        color(Nat.0, Nat.2) = color(Nat.0, Nat.3)
        color(Nat.0, Nat.3) = color(Nat.0, Nat.4)
        good_four_1234
        good_four(Nat.1, Nat.2, Nat.3, Nat.4)
        four_edges_from_zero_share_color_intro(color, Nat.1, Nat.2, Nat.3, Nat.4)
        four_edges_from_zero_share_color(color)
        }
        } else {
        if color(Nat.0, Nat.8) {
        color(Nat.0, Nat.1) = color(Nat.0, Nat.2)
        color(Nat.0, Nat.2) = color(Nat.0, Nat.3)
        color(Nat.0, Nat.3) = color(Nat.0, Nat.4)
        good_four_1234
        good_four(Nat.1, Nat.2, Nat.3, Nat.4)
        four_edges_from_zero_share_color_intro(color, Nat.1, Nat.2, Nat.3, Nat.4)
        four_edges_from_zero_share_color(color)
        } else {
        color(Nat.0, Nat.1) = color(Nat.0, Nat.2)
        color(Nat.0, Nat.2) = color(Nat.0, Nat.3)
        color(Nat.0, Nat.3) = color(Nat.0, Nat.4)
        good_four_1234
        good_four(Nat.1, Nat.2, Nat.3, Nat.4)
        four_edges_from_zero_share_color_intro(color, Nat.1, Nat.2, Nat.3, Nat.4)
        four_edges_from_zero_share_color(color)
        }
        }
        }
        } else {
        if color(Nat.0, Nat.6) {
        if color(Nat.0, Nat.7) {
        if color(Nat.0, Nat.8) {
        color(Nat.0, Nat.1) = color(Nat.0, Nat.2)
        color(Nat.0, Nat.2) = color(Nat.0, Nat.3)
        color(Nat.0, Nat.3) = color(Nat.0, Nat.4)
        good_four_1234
        good_four(Nat.1, Nat.2, Nat.3, Nat.4)
        four_edges_from_zero_share_color_intro(color, Nat.1, Nat.2, Nat.3, Nat.4)
        four_edges_from_zero_share_color(color)
        } else {
        color(Nat.0, Nat.1) = color(Nat.0, Nat.2)
        color(Nat.0, Nat.2) = color(Nat.0, Nat.3)
        color(Nat.0, Nat.3) = color(Nat.0, Nat.4)
        good_four_1234
        good_four(Nat.1, Nat.2, Nat.3, Nat.4)
        four_edges_from_zero_share_color_intro(color, Nat.1, Nat.2, Nat.3, Nat.4)
        four_edges_from_zero_share_color(color)
        }
        } else {
        if color(Nat.0, Nat.8) {
        color(Nat.0, Nat.1) = color(Nat.0, Nat.2)
        color(Nat.0, Nat.2) = color(Nat.0, Nat.3)
        color(Nat.0, Nat.3) = color(Nat.0, Nat.4)
        good_four_1234
        good_four(Nat.1, Nat.2, Nat.3, Nat.4)
        four_edges_from_zero_share_color_intro(color, Nat.1, Nat.2, Nat.3, Nat.4)
        four_edges_from_zero_share_color(color)
        } else {
        color(Nat.0, Nat.1) = color(Nat.0, Nat.2)
        color(Nat.0, Nat.2) = color(Nat.0, Nat.3)
        color(Nat.0, Nat.3) = color(Nat.0, Nat.4)
        good_four_1234
        good_four(Nat.1, Nat.2, Nat.3, Nat.4)
        four_edges_from_zero_share_color_intro(color, Nat.1, Nat.2, Nat.3, Nat.4)
        four_edges_from_zero_share_color(color)
        }
        }
        } else {
        if color(Nat.0, Nat.7) {
        if color(Nat.0, Nat.8) {
        color(Nat.0, Nat.1) = color(Nat.0, Nat.2)
        color(Nat.0, Nat.2) = color(Nat.0, Nat.3)
        color(Nat.0, Nat.3) = color(Nat.0, Nat.4)
        good_four_1234
        good_four(Nat.1, Nat.2, Nat.3, Nat.4)
        four_edges_from_zero_share_color_intro(color, Nat.1, Nat.2, Nat.3, Nat.4)
        four_edges_from_zero_share_color(color)
        } else {
        color(Nat.0, Nat.1) = color(Nat.0, Nat.2)
        color(Nat.0, Nat.2) = color(Nat.0, Nat.3)
        color(Nat.0, Nat.3) = color(Nat.0, Nat.4)
        good_four_1234
        good_four(Nat.1, Nat.2, Nat.3, Nat.4)
        four_edges_from_zero_share_color_intro(color, Nat.1, Nat.2, Nat.3, Nat.4)
        four_edges_from_zero_share_color(color)
        }
        } else {
        if color(Nat.0, Nat.8) {
        color(Nat.0, Nat.1) = color(Nat.0, Nat.2)
        color(Nat.0, Nat.2) = color(Nat.0, Nat.3)
        color(Nat.0, Nat.3) = color(Nat.0, Nat.4)
        good_four_1234
        good_four(Nat.1, Nat.2, Nat.3, Nat.4)
        four_edges_from_zero_share_color_intro(color, Nat.1, Nat.2, Nat.3, Nat.4)
        four_edges_from_zero_share_color(color)
        } else {
        color(Nat.0, Nat.1) = color(Nat.0, Nat.2)
        color(Nat.0, Nat.2) = color(Nat.0, Nat.3)
        color(Nat.0, Nat.3) = color(Nat.0, Nat.4)
        good_four_1234
        good_four(Nat.1, Nat.2, Nat.3, Nat.4)
        four_edges_from_zero_share_color_intro(color, Nat.1, Nat.2, Nat.3, Nat.4)
        four_edges_from_zero_share_color(color)
        }
        }
        }
        }
        } else {
        if color(Nat.0, Nat.5) {
        if color(Nat.0, Nat.6) {
        if color(Nat.0, Nat.7) {
        if color(Nat.0, Nat.8) {
        color(Nat.0, Nat.1) = color(Nat.0, Nat.2)
        color(Nat.0, Nat.2) = color(Nat.0, Nat.3)
        color(Nat.0, Nat.3) = color(Nat.0, Nat.5)
        good_four_1235
        good_four(Nat.1, Nat.2, Nat.3, Nat.5)
        four_edges_from_zero_share_color_intro(color, Nat.1, Nat.2, Nat.3, Nat.5)
        four_edges_from_zero_share_color(color)
        } else {
        color(Nat.0, Nat.1) = color(Nat.0, Nat.2)
        color(Nat.0, Nat.2) = color(Nat.0, Nat.3)
        color(Nat.0, Nat.3) = color(Nat.0, Nat.5)
        good_four_1235
        good_four(Nat.1, Nat.2, Nat.3, Nat.5)
        four_edges_from_zero_share_color_intro(color, Nat.1, Nat.2, Nat.3, Nat.5)
        four_edges_from_zero_share_color(color)
        }
        } else {
        if color(Nat.0, Nat.8) {
        color(Nat.0, Nat.1) = color(Nat.0, Nat.2)
        color(Nat.0, Nat.2) = color(Nat.0, Nat.3)
        color(Nat.0, Nat.3) = color(Nat.0, Nat.5)
        good_four_1235
        good_four(Nat.1, Nat.2, Nat.3, Nat.5)
        four_edges_from_zero_share_color_intro(color, Nat.1, Nat.2, Nat.3, Nat.5)
        four_edges_from_zero_share_color(color)
        } else {
        color(Nat.0, Nat.1) = color(Nat.0, Nat.2)
        color(Nat.0, Nat.2) = color(Nat.0, Nat.3)
        color(Nat.0, Nat.3) = color(Nat.0, Nat.5)
        good_four_1235
        good_four(Nat.1, Nat.2, Nat.3, Nat.5)
        four_edges_from_zero_share_color_intro(color, Nat.1, Nat.2, Nat.3, Nat.5)
        four_edges_from_zero_share_color(color)
        }
        }
        } else {
        if color(Nat.0, Nat.7) {
        if color(Nat.0, Nat.8) {
        color(Nat.0, Nat.1) = color(Nat.0, Nat.2)
        color(Nat.0, Nat.2) = color(Nat.0, Nat.3)
        color(Nat.0, Nat.3) = color(Nat.0, Nat.5)
        good_four_1235
        good_four(Nat.1, Nat.2, Nat.3, Nat.5)
        four_edges_from_zero_share_color_intro(color, Nat.1, Nat.2, Nat.3, Nat.5)
        four_edges_from_zero_share_color(color)
        } else {
        color(Nat.0, Nat.1) = color(Nat.0, Nat.2)
        color(Nat.0, Nat.2) = color(Nat.0, Nat.3)
        color(Nat.0, Nat.3) = color(Nat.0, Nat.5)
        good_four_1235
        good_four(Nat.1, Nat.2, Nat.3, Nat.5)
        four_edges_from_zero_share_color_intro(color, Nat.1, Nat.2, Nat.3, Nat.5)
        four_edges_from_zero_share_color(color)
        }
        } else {
        if color(Nat.0, Nat.8) {
        color(Nat.0, Nat.1) = color(Nat.0, Nat.2)
        color(Nat.0, Nat.2) = color(Nat.0, Nat.3)
        color(Nat.0, Nat.3) = color(Nat.0, Nat.5)
        good_four_1235
        good_four(Nat.1, Nat.2, Nat.3, Nat.5)
        four_edges_from_zero_share_color_intro(color, Nat.1, Nat.2, Nat.3, Nat.5)
        four_edges_from_zero_share_color(color)
        } else {
        color(Nat.0, Nat.1) = color(Nat.0, Nat.2)
        color(Nat.0, Nat.2) = color(Nat.0, Nat.3)
        color(Nat.0, Nat.3) = color(Nat.0, Nat.5)
        good_four_1235
        good_four(Nat.1, Nat.2, Nat.3, Nat.5)
        four_edges_from_zero_share_color_intro(color, Nat.1, Nat.2, Nat.3, Nat.5)
        four_edges_from_zero_share_color(color)
        }
        }
        }
        } else {
        if color(Nat.0, Nat.6) {
        if color(Nat.0, Nat.7) {
        if color(Nat.0, Nat.8) {
        color(Nat.0, Nat.1) = color(Nat.0, Nat.2)
        color(Nat.0, Nat.2) = color(Nat.0, Nat.3)
        color(Nat.0, Nat.3) = color(Nat.0, Nat.6)
        good_four_1236
        good_four(Nat.1, Nat.2, Nat.3, Nat.6)
        four_edges_from_zero_share_color_intro(color, Nat.1, Nat.2, Nat.3, Nat.6)
        four_edges_from_zero_share_color(color)
        } else {
        color(Nat.0, Nat.1) = color(Nat.0, Nat.2)
        color(Nat.0, Nat.2) = color(Nat.0, Nat.3)
        color(Nat.0, Nat.3) = color(Nat.0, Nat.6)
        good_four_1236
        good_four(Nat.1, Nat.2, Nat.3, Nat.6)
        four_edges_from_zero_share_color_intro(color, Nat.1, Nat.2, Nat.3, Nat.6)
        four_edges_from_zero_share_color(color)
        }
        } else {
        if color(Nat.0, Nat.8) {
        color(Nat.0, Nat.1) = color(Nat.0, Nat.2)
        color(Nat.0, Nat.2) = color(Nat.0, Nat.3)
        color(Nat.0, Nat.3) = color(Nat.0, Nat.6)
        good_four_1236
        good_four(Nat.1, Nat.2, Nat.3, Nat.6)
        four_edges_from_zero_share_color_intro(color, Nat.1, Nat.2, Nat.3, Nat.6)
        four_edges_from_zero_share_color(color)
        } else {
        color(Nat.0, Nat.1) = color(Nat.0, Nat.2)
        color(Nat.0, Nat.2) = color(Nat.0, Nat.3)
        color(Nat.0, Nat.3) = color(Nat.0, Nat.6)
        good_four_1236
        good_four(Nat.1, Nat.2, Nat.3, Nat.6)
        four_edges_from_zero_share_color_intro(color, Nat.1, Nat.2, Nat.3, Nat.6)
        four_edges_from_zero_share_color(color)
        }
        }
        } else {
        if color(Nat.0, Nat.7) {
        if color(Nat.0, Nat.8) {
        color(Nat.0, Nat.1) = color(Nat.0, Nat.2)
        color(Nat.0, Nat.2) = color(Nat.0, Nat.3)
        color(Nat.0, Nat.3) = color(Nat.0, Nat.7)
        good_four_1237
        good_four(Nat.1, Nat.2, Nat.3, Nat.7)
        four_edges_from_zero_share_color_intro(color, Nat.1, Nat.2, Nat.3, Nat.7)
        four_edges_from_zero_share_color(color)
        } else {
        color(Nat.0, Nat.1) = color(Nat.0, Nat.2)
        color(Nat.0, Nat.2) = color(Nat.0, Nat.3)
        color(Nat.0, Nat.3) = color(Nat.0, Nat.7)
        good_four_1237
        good_four(Nat.1, Nat.2, Nat.3, Nat.7)
        four_edges_from_zero_share_color_intro(color, Nat.1, Nat.2, Nat.3, Nat.7)
        four_edges_from_zero_share_color(color)
        }
        } else {
        if color(Nat.0, Nat.8) {
        color(Nat.0, Nat.1) = color(Nat.0, Nat.2)
        color(Nat.0, Nat.2) = color(Nat.0, Nat.3)
        color(Nat.0, Nat.3) = color(Nat.0, Nat.8)
        good_four_1238
        good_four(Nat.1, Nat.2, Nat.3, Nat.8)
        four_edges_from_zero_share_color_intro(color, Nat.1, Nat.2, Nat.3, Nat.8)
        four_edges_from_zero_share_color(color)
        } else {
        color(Nat.0, Nat.4) = color(Nat.0, Nat.5)
        color(Nat.0, Nat.5) = color(Nat.0, Nat.6)
        color(Nat.0, Nat.6) = color(Nat.0, Nat.7)
        good_four_4567
        good_four(Nat.4, Nat.5, Nat.6, Nat.7)
        four_edges_from_zero_share_color_intro(color, Nat.4, Nat.5, Nat.6, Nat.7)
        four_edges_from_zero_share_color(color)
        }
        }
        }
        }
        }
        } else {
        if color(Nat.0, Nat.4) {
        if color(Nat.0, Nat.5) {
        if color(Nat.0, Nat.6) {
        if color(Nat.0, Nat.7) {
        if color(Nat.0, Nat.8) {
        color(Nat.0, Nat.1) = color(Nat.0, Nat.2)
        color(Nat.0, Nat.2) = color(Nat.0, Nat.4)
        color(Nat.0, Nat.4) = color(Nat.0, Nat.5)
        good_four_1245
        good_four(Nat.1, Nat.2, Nat.4, Nat.5)
        four_edges_from_zero_share_color_intro(color, Nat.1, Nat.2, Nat.4, Nat.5)
        four_edges_from_zero_share_color(color)
        } else {
        color(Nat.0, Nat.1) = color(Nat.0, Nat.2)
        color(Nat.0, Nat.2) = color(Nat.0, Nat.4)
        color(Nat.0, Nat.4) = color(Nat.0, Nat.5)
        good_four_1245
        good_four(Nat.1, Nat.2, Nat.4, Nat.5)
        four_edges_from_zero_share_color_intro(color, Nat.1, Nat.2, Nat.4, Nat.5)
        four_edges_from_zero_share_color(color)
        }
        } else {
        if color(Nat.0, Nat.8) {
        color(Nat.0, Nat.1) = color(Nat.0, Nat.2)
        color(Nat.0, Nat.2) = color(Nat.0, Nat.4)
        color(Nat.0, Nat.4) = color(Nat.0, Nat.5)
        good_four_1245
        good_four(Nat.1, Nat.2, Nat.4, Nat.5)
        four_edges_from_zero_share_color_intro(color, Nat.1, Nat.2, Nat.4, Nat.5)
        four_edges_from_zero_share_color(color)
        } else {
        color(Nat.0, Nat.1) = color(Nat.0, Nat.2)
        color(Nat.0, Nat.2) = color(Nat.0, Nat.4)
        color(Nat.0, Nat.4) = color(Nat.0, Nat.5)
        good_four_1245
        good_four(Nat.1, Nat.2, Nat.4, Nat.5)
        four_edges_from_zero_share_color_intro(color, Nat.1, Nat.2, Nat.4, Nat.5)
        four_edges_from_zero_share_color(color)
        }
        }
        } else {
        if color(Nat.0, Nat.7) {
        if color(Nat.0, Nat.8) {
        color(Nat.0, Nat.1) = color(Nat.0, Nat.2)
        color(Nat.0, Nat.2) = color(Nat.0, Nat.4)
        color(Nat.0, Nat.4) = color(Nat.0, Nat.5)
        good_four_1245
        good_four(Nat.1, Nat.2, Nat.4, Nat.5)
        four_edges_from_zero_share_color_intro(color, Nat.1, Nat.2, Nat.4, Nat.5)
        four_edges_from_zero_share_color(color)
        } else {
        color(Nat.0, Nat.1) = color(Nat.0, Nat.2)
        color(Nat.0, Nat.2) = color(Nat.0, Nat.4)
        color(Nat.0, Nat.4) = color(Nat.0, Nat.5)
        good_four_1245
        good_four(Nat.1, Nat.2, Nat.4, Nat.5)
        four_edges_from_zero_share_color_intro(color, Nat.1, Nat.2, Nat.4, Nat.5)
        four_edges_from_zero_share_color(color)
        }
        } else {
        if color(Nat.0, Nat.8) {
        color(Nat.0, Nat.1) = color(Nat.0, Nat.2)
        color(Nat.0, Nat.2) = color(Nat.0, Nat.4)
        color(Nat.0, Nat.4) = color(Nat.0, Nat.5)
        good_four_1245
        good_four(Nat.1, Nat.2, Nat.4, Nat.5)
        four_edges_from_zero_share_color_intro(color, Nat.1, Nat.2, Nat.4, Nat.5)
        four_edges_from_zero_share_color(color)
        } else {
        color(Nat.0, Nat.1) = color(Nat.0, Nat.2)
        color(Nat.0, Nat.2) = color(Nat.0, Nat.4)
        color(Nat.0, Nat.4) = color(Nat.0, Nat.5)
        good_four_1245
        good_four(Nat.1, Nat.2, Nat.4, Nat.5)
        four_edges_from_zero_share_color_intro(color, Nat.1, Nat.2, Nat.4, Nat.5)
        four_edges_from_zero_share_color(color)
        }
        }
        }
        } else {
        if color(Nat.0, Nat.6) {
        if color(Nat.0, Nat.7) {
        if color(Nat.0, Nat.8) {
        color(Nat.0, Nat.1) = color(Nat.0, Nat.2)
        color(Nat.0, Nat.2) = color(Nat.0, Nat.4)
        color(Nat.0, Nat.4) = color(Nat.0, Nat.6)
        good_four_1246
        good_four(Nat.1, Nat.2, Nat.4, Nat.6)
        four_edges_from_zero_share_color_intro(color, Nat.1, Nat.2, Nat.4, Nat.6)
        four_edges_from_zero_share_color(color)
        } else {
        color(Nat.0, Nat.1) = color(Nat.0, Nat.2)
        color(Nat.0, Nat.2) = color(Nat.0, Nat.4)
        color(Nat.0, Nat.4) = color(Nat.0, Nat.6)
        good_four_1246
        good_four(Nat.1, Nat.2, Nat.4, Nat.6)
        four_edges_from_zero_share_color_intro(color, Nat.1, Nat.2, Nat.4, Nat.6)
        four_edges_from_zero_share_color(color)
        }
        } else {
        if color(Nat.0, Nat.8) {
        color(Nat.0, Nat.1) = color(Nat.0, Nat.2)
        color(Nat.0, Nat.2) = color(Nat.0, Nat.4)
        color(Nat.0, Nat.4) = color(Nat.0, Nat.6)
        good_four_1246
        good_four(Nat.1, Nat.2, Nat.4, Nat.6)
        four_edges_from_zero_share_color_intro(color, Nat.1, Nat.2, Nat.4, Nat.6)
        four_edges_from_zero_share_color(color)
        } else {
        color(Nat.0, Nat.1) = color(Nat.0, Nat.2)
        color(Nat.0, Nat.2) = color(Nat.0, Nat.4)
        color(Nat.0, Nat.4) = color(Nat.0, Nat.6)
        good_four_1246
        good_four(Nat.1, Nat.2, Nat.4, Nat.6)
        four_edges_from_zero_share_color_intro(color, Nat.1, Nat.2, Nat.4, Nat.6)
        four_edges_from_zero_share_color(color)
        }
        }
        } else {
        if color(Nat.0, Nat.7) {
        if color(Nat.0, Nat.8) {
        color(Nat.0, Nat.1) = color(Nat.0, Nat.2)
        color(Nat.0, Nat.2) = color(Nat.0, Nat.4)
        color(Nat.0, Nat.4) = color(Nat.0, Nat.7)
        good_four_1247
        good_four(Nat.1, Nat.2, Nat.4, Nat.7)
        four_edges_from_zero_share_color_intro(color, Nat.1, Nat.2, Nat.4, Nat.7)
        four_edges_from_zero_share_color(color)
        } else {
        color(Nat.0, Nat.1) = color(Nat.0, Nat.2)
        color(Nat.0, Nat.2) = color(Nat.0, Nat.4)
        color(Nat.0, Nat.4) = color(Nat.0, Nat.7)
        good_four_1247
        good_four(Nat.1, Nat.2, Nat.4, Nat.7)
        four_edges_from_zero_share_color_intro(color, Nat.1, Nat.2, Nat.4, Nat.7)
        four_edges_from_zero_share_color(color)
        }
        } else {
        if color(Nat.0, Nat.8) {
        color(Nat.0, Nat.1) = color(Nat.0, Nat.2)
        color(Nat.0, Nat.2) = color(Nat.0, Nat.4)
        color(Nat.0, Nat.4) = color(Nat.0, Nat.8)
        good_four_1248
        good_four(Nat.1, Nat.2, Nat.4, Nat.8)
        four_edges_from_zero_share_color_intro(color, Nat.1, Nat.2, Nat.4, Nat.8)
        four_edges_from_zero_share_color(color)
        } else {
        color(Nat.0, Nat.3) = color(Nat.0, Nat.5)
        color(Nat.0, Nat.5) = color(Nat.0, Nat.6)
        color(Nat.0, Nat.6) = color(Nat.0, Nat.7)
        good_four_3567
        good_four(Nat.3, Nat.5, Nat.6, Nat.7)
        four_edges_from_zero_share_color_intro(color, Nat.3, Nat.5, Nat.6, Nat.7)
        four_edges_from_zero_share_color(color)
        }
        }
        }
        }
        } else {
        if color(Nat.0, Nat.5) {
        if color(Nat.0, Nat.6) {
        if color(Nat.0, Nat.7) {
        if color(Nat.0, Nat.8) {
        color(Nat.0, Nat.1) = color(Nat.0, Nat.2)
        color(Nat.0, Nat.2) = color(Nat.0, Nat.5)
        color(Nat.0, Nat.5) = color(Nat.0, Nat.6)
        good_four_1256
        good_four(Nat.1, Nat.2, Nat.5, Nat.6)
        four_edges_from_zero_share_color_intro(color, Nat.1, Nat.2, Nat.5, Nat.6)
        four_edges_from_zero_share_color(color)
        } else {
        color(Nat.0, Nat.1) = color(Nat.0, Nat.2)
        color(Nat.0, Nat.2) = color(Nat.0, Nat.5)
        color(Nat.0, Nat.5) = color(Nat.0, Nat.6)
        good_four_1256
        good_four(Nat.1, Nat.2, Nat.5, Nat.6)
        four_edges_from_zero_share_color_intro(color, Nat.1, Nat.2, Nat.5, Nat.6)
        four_edges_from_zero_share_color(color)
        }
        } else {
        if color(Nat.0, Nat.8) {
        color(Nat.0, Nat.1) = color(Nat.0, Nat.2)
        color(Nat.0, Nat.2) = color(Nat.0, Nat.5)
        color(Nat.0, Nat.5) = color(Nat.0, Nat.6)
        good_four_1256
        good_four(Nat.1, Nat.2, Nat.5, Nat.6)
        four_edges_from_zero_share_color_intro(color, Nat.1, Nat.2, Nat.5, Nat.6)
        four_edges_from_zero_share_color(color)
        } else {
        color(Nat.0, Nat.1) = color(Nat.0, Nat.2)
        color(Nat.0, Nat.2) = color(Nat.0, Nat.5)
        color(Nat.0, Nat.5) = color(Nat.0, Nat.6)
        good_four_1256
        good_four(Nat.1, Nat.2, Nat.5, Nat.6)
        four_edges_from_zero_share_color_intro(color, Nat.1, Nat.2, Nat.5, Nat.6)
        four_edges_from_zero_share_color(color)
        }
        }
        } else {
        if color(Nat.0, Nat.7) {
        if color(Nat.0, Nat.8) {
        color(Nat.0, Nat.1) = color(Nat.0, Nat.2)
        color(Nat.0, Nat.2) = color(Nat.0, Nat.5)
        color(Nat.0, Nat.5) = color(Nat.0, Nat.7)
        good_four_1257
        good_four(Nat.1, Nat.2, Nat.5, Nat.7)
        four_edges_from_zero_share_color_intro(color, Nat.1, Nat.2, Nat.5, Nat.7)
        four_edges_from_zero_share_color(color)
        } else {
        color(Nat.0, Nat.1) = color(Nat.0, Nat.2)
        color(Nat.0, Nat.2) = color(Nat.0, Nat.5)
        color(Nat.0, Nat.5) = color(Nat.0, Nat.7)
        good_four_1257
        good_four(Nat.1, Nat.2, Nat.5, Nat.7)
        four_edges_from_zero_share_color_intro(color, Nat.1, Nat.2, Nat.5, Nat.7)
        four_edges_from_zero_share_color(color)
        }
        } else {
        if color(Nat.0, Nat.8) {
        color(Nat.0, Nat.1) = color(Nat.0, Nat.2)
        color(Nat.0, Nat.2) = color(Nat.0, Nat.5)
        color(Nat.0, Nat.5) = color(Nat.0, Nat.8)
        good_four_1258
        good_four(Nat.1, Nat.2, Nat.5, Nat.8)
        four_edges_from_zero_share_color_intro(color, Nat.1, Nat.2, Nat.5, Nat.8)
        four_edges_from_zero_share_color(color)
        } else {
        color(Nat.0, Nat.3) = color(Nat.0, Nat.4)
        color(Nat.0, Nat.4) = color(Nat.0, Nat.6)
        color(Nat.0, Nat.6) = color(Nat.0, Nat.7)
        good_four_3467
        good_four(Nat.3, Nat.4, Nat.6, Nat.7)
        four_edges_from_zero_share_color_intro(color, Nat.3, Nat.4, Nat.6, Nat.7)
        four_edges_from_zero_share_color(color)
        }
        }
        }
        } else {
        if color(Nat.0, Nat.6) {
        if color(Nat.0, Nat.7) {
        if color(Nat.0, Nat.8) {
        color(Nat.0, Nat.1) = color(Nat.0, Nat.2)
        color(Nat.0, Nat.2) = color(Nat.0, Nat.6)
        color(Nat.0, Nat.6) = color(Nat.0, Nat.7)
        good_four_1267
        good_four(Nat.1, Nat.2, Nat.6, Nat.7)
        four_edges_from_zero_share_color_intro(color, Nat.1, Nat.2, Nat.6, Nat.7)
        four_edges_from_zero_share_color(color)
        } else {
        color(Nat.0, Nat.1) = color(Nat.0, Nat.2)
        color(Nat.0, Nat.2) = color(Nat.0, Nat.6)
        color(Nat.0, Nat.6) = color(Nat.0, Nat.7)
        good_four_1267
        good_four(Nat.1, Nat.2, Nat.6, Nat.7)
        four_edges_from_zero_share_color_intro(color, Nat.1, Nat.2, Nat.6, Nat.7)
        four_edges_from_zero_share_color(color)
        }
        } else {
        if color(Nat.0, Nat.8) {
        color(Nat.0, Nat.1) = color(Nat.0, Nat.2)
        color(Nat.0, Nat.2) = color(Nat.0, Nat.6)
        color(Nat.0, Nat.6) = color(Nat.0, Nat.8)
        good_four_1268
        good_four(Nat.1, Nat.2, Nat.6, Nat.8)
        four_edges_from_zero_share_color_intro(color, Nat.1, Nat.2, Nat.6, Nat.8)
        four_edges_from_zero_share_color(color)
        } else {
        color(Nat.0, Nat.3) = color(Nat.0, Nat.4)
        color(Nat.0, Nat.4) = color(Nat.0, Nat.5)
        color(Nat.0, Nat.5) = color(Nat.0, Nat.7)
        good_four_3457
        good_four(Nat.3, Nat.4, Nat.5, Nat.7)
        four_edges_from_zero_share_color_intro(color, Nat.3, Nat.4, Nat.5, Nat.7)
        four_edges_from_zero_share_color(color)
        }
        }
        } else {
        if color(Nat.0, Nat.7) {
        if color(Nat.0, Nat.8) {
        color(Nat.0, Nat.1) = color(Nat.0, Nat.2)
        color(Nat.0, Nat.2) = color(Nat.0, Nat.7)
        color(Nat.0, Nat.7) = color(Nat.0, Nat.8)
        good_four_1278
        good_four(Nat.1, Nat.2, Nat.7, Nat.8)
        four_edges_from_zero_share_color_intro(color, Nat.1, Nat.2, Nat.7, Nat.8)
        four_edges_from_zero_share_color(color)
        } else {
        color(Nat.0, Nat.3) = color(Nat.0, Nat.4)
        color(Nat.0, Nat.4) = color(Nat.0, Nat.5)
        color(Nat.0, Nat.5) = color(Nat.0, Nat.6)
        good_four_3456
        good_four(Nat.3, Nat.4, Nat.5, Nat.6)
        four_edges_from_zero_share_color_intro(color, Nat.3, Nat.4, Nat.5, Nat.6)
        four_edges_from_zero_share_color(color)
        }
        } else {
        if color(Nat.0, Nat.8) {
        color(Nat.0, Nat.3) = color(Nat.0, Nat.4)
        color(Nat.0, Nat.4) = color(Nat.0, Nat.5)
        color(Nat.0, Nat.5) = color(Nat.0, Nat.6)
        good_four_3456
        good_four(Nat.3, Nat.4, Nat.5, Nat.6)
        four_edges_from_zero_share_color_intro(color, Nat.3, Nat.4, Nat.5, Nat.6)
        four_edges_from_zero_share_color(color)
        } else {
        color(Nat.0, Nat.3) = color(Nat.0, Nat.4)
        color(Nat.0, Nat.4) = color(Nat.0, Nat.5)
        color(Nat.0, Nat.5) = color(Nat.0, Nat.6)
        good_four_3456
        good_four(Nat.3, Nat.4, Nat.5, Nat.6)
        four_edges_from_zero_share_color_intro(color, Nat.3, Nat.4, Nat.5, Nat.6)
        four_edges_from_zero_share_color(color)
        }
        }
        }
        }
        }
        }
        } else {
        if color(Nat.0, Nat.3) {
        if color(Nat.0, Nat.4) {
        if color(Nat.0, Nat.5) {
        if color(Nat.0, Nat.6) {
        if color(Nat.0, Nat.7) {
        if color(Nat.0, Nat.8) {
        color(Nat.0, Nat.1) = color(Nat.0, Nat.3)
        color(Nat.0, Nat.3) = color(Nat.0, Nat.4)
        color(Nat.0, Nat.4) = color(Nat.0, Nat.5)
        good_four_1345
        good_four(Nat.1, Nat.3, Nat.4, Nat.5)
        four_edges_from_zero_share_color_intro(color, Nat.1, Nat.3, Nat.4, Nat.5)
        four_edges_from_zero_share_color(color)
        } else {
        color(Nat.0, Nat.1) = color(Nat.0, Nat.3)
        color(Nat.0, Nat.3) = color(Nat.0, Nat.4)
        color(Nat.0, Nat.4) = color(Nat.0, Nat.5)
        good_four_1345
        good_four(Nat.1, Nat.3, Nat.4, Nat.5)
        four_edges_from_zero_share_color_intro(color, Nat.1, Nat.3, Nat.4, Nat.5)
        four_edges_from_zero_share_color(color)
        }
        } else {
        if color(Nat.0, Nat.8) {
        color(Nat.0, Nat.1) = color(Nat.0, Nat.3)
        color(Nat.0, Nat.3) = color(Nat.0, Nat.4)
        color(Nat.0, Nat.4) = color(Nat.0, Nat.5)
        good_four_1345
        good_four(Nat.1, Nat.3, Nat.4, Nat.5)
        four_edges_from_zero_share_color_intro(color, Nat.1, Nat.3, Nat.4, Nat.5)
        four_edges_from_zero_share_color(color)
        } else {
        color(Nat.0, Nat.1) = color(Nat.0, Nat.3)
        color(Nat.0, Nat.3) = color(Nat.0, Nat.4)
        color(Nat.0, Nat.4) = color(Nat.0, Nat.5)
        good_four_1345
        good_four(Nat.1, Nat.3, Nat.4, Nat.5)
        four_edges_from_zero_share_color_intro(color, Nat.1, Nat.3, Nat.4, Nat.5)
        four_edges_from_zero_share_color(color)
        }
        }
        } else {
        if color(Nat.0, Nat.7) {
        if color(Nat.0, Nat.8) {
        color(Nat.0, Nat.1) = color(Nat.0, Nat.3)
        color(Nat.0, Nat.3) = color(Nat.0, Nat.4)
        color(Nat.0, Nat.4) = color(Nat.0, Nat.5)
        good_four_1345
        good_four(Nat.1, Nat.3, Nat.4, Nat.5)
        four_edges_from_zero_share_color_intro(color, Nat.1, Nat.3, Nat.4, Nat.5)
        four_edges_from_zero_share_color(color)
        } else {
        color(Nat.0, Nat.1) = color(Nat.0, Nat.3)
        color(Nat.0, Nat.3) = color(Nat.0, Nat.4)
        color(Nat.0, Nat.4) = color(Nat.0, Nat.5)
        good_four_1345
        good_four(Nat.1, Nat.3, Nat.4, Nat.5)
        four_edges_from_zero_share_color_intro(color, Nat.1, Nat.3, Nat.4, Nat.5)
        four_edges_from_zero_share_color(color)
        }
        } else {
        if color(Nat.0, Nat.8) {
        color(Nat.0, Nat.1) = color(Nat.0, Nat.3)
        color(Nat.0, Nat.3) = color(Nat.0, Nat.4)
        color(Nat.0, Nat.4) = color(Nat.0, Nat.5)
        good_four_1345
        good_four(Nat.1, Nat.3, Nat.4, Nat.5)
        four_edges_from_zero_share_color_intro(color, Nat.1, Nat.3, Nat.4, Nat.5)
        four_edges_from_zero_share_color(color)
        } else {
        color(Nat.0, Nat.1) = color(Nat.0, Nat.3)
        color(Nat.0, Nat.3) = color(Nat.0, Nat.4)
        color(Nat.0, Nat.4) = color(Nat.0, Nat.5)
        good_four_1345
        good_four(Nat.1, Nat.3, Nat.4, Nat.5)
        four_edges_from_zero_share_color_intro(color, Nat.1, Nat.3, Nat.4, Nat.5)
        four_edges_from_zero_share_color(color)
        }
        }
        }
        } else {
        if color(Nat.0, Nat.6) {
        if color(Nat.0, Nat.7) {
        if color(Nat.0, Nat.8) {
        color(Nat.0, Nat.1) = color(Nat.0, Nat.3)
        color(Nat.0, Nat.3) = color(Nat.0, Nat.4)
        color(Nat.0, Nat.4) = color(Nat.0, Nat.6)
        good_four_1346
        good_four(Nat.1, Nat.3, Nat.4, Nat.6)
        four_edges_from_zero_share_color_intro(color, Nat.1, Nat.3, Nat.4, Nat.6)
        four_edges_from_zero_share_color(color)
        } else {
        color(Nat.0, Nat.1) = color(Nat.0, Nat.3)
        color(Nat.0, Nat.3) = color(Nat.0, Nat.4)
        color(Nat.0, Nat.4) = color(Nat.0, Nat.6)
        good_four_1346
        good_four(Nat.1, Nat.3, Nat.4, Nat.6)
        four_edges_from_zero_share_color_intro(color, Nat.1, Nat.3, Nat.4, Nat.6)
        four_edges_from_zero_share_color(color)
        }
        } else {
        if color(Nat.0, Nat.8) {
        color(Nat.0, Nat.1) = color(Nat.0, Nat.3)
        color(Nat.0, Nat.3) = color(Nat.0, Nat.4)
        color(Nat.0, Nat.4) = color(Nat.0, Nat.6)
        good_four_1346
        good_four(Nat.1, Nat.3, Nat.4, Nat.6)
        four_edges_from_zero_share_color_intro(color, Nat.1, Nat.3, Nat.4, Nat.6)
        four_edges_from_zero_share_color(color)
        } else {
        color(Nat.0, Nat.1) = color(Nat.0, Nat.3)
        color(Nat.0, Nat.3) = color(Nat.0, Nat.4)
        color(Nat.0, Nat.4) = color(Nat.0, Nat.6)
        good_four_1346
        good_four(Nat.1, Nat.3, Nat.4, Nat.6)
        four_edges_from_zero_share_color_intro(color, Nat.1, Nat.3, Nat.4, Nat.6)
        four_edges_from_zero_share_color(color)
        }
        }
        } else {
        if color(Nat.0, Nat.7) {
        if color(Nat.0, Nat.8) {
        color(Nat.0, Nat.1) = color(Nat.0, Nat.3)
        color(Nat.0, Nat.3) = color(Nat.0, Nat.4)
        color(Nat.0, Nat.4) = color(Nat.0, Nat.7)
        good_four_1347
        good_four(Nat.1, Nat.3, Nat.4, Nat.7)
        four_edges_from_zero_share_color_intro(color, Nat.1, Nat.3, Nat.4, Nat.7)
        four_edges_from_zero_share_color(color)
        } else {
        color(Nat.0, Nat.1) = color(Nat.0, Nat.3)
        color(Nat.0, Nat.3) = color(Nat.0, Nat.4)
        color(Nat.0, Nat.4) = color(Nat.0, Nat.7)
        good_four_1347
        good_four(Nat.1, Nat.3, Nat.4, Nat.7)
        four_edges_from_zero_share_color_intro(color, Nat.1, Nat.3, Nat.4, Nat.7)
        four_edges_from_zero_share_color(color)
        }
        } else {
        if color(Nat.0, Nat.8) {
        color(Nat.0, Nat.1) = color(Nat.0, Nat.3)
        color(Nat.0, Nat.3) = color(Nat.0, Nat.4)
        color(Nat.0, Nat.4) = color(Nat.0, Nat.8)
        good_four_1348
        good_four(Nat.1, Nat.3, Nat.4, Nat.8)
        four_edges_from_zero_share_color_intro(color, Nat.1, Nat.3, Nat.4, Nat.8)
        four_edges_from_zero_share_color(color)
        } else {
        color(Nat.0, Nat.2) = color(Nat.0, Nat.5)
        color(Nat.0, Nat.5) = color(Nat.0, Nat.6)
        color(Nat.0, Nat.6) = color(Nat.0, Nat.7)
        good_four_2567
        good_four(Nat.2, Nat.5, Nat.6, Nat.7)
        four_edges_from_zero_share_color_intro(color, Nat.2, Nat.5, Nat.6, Nat.7)
        four_edges_from_zero_share_color(color)
        }
        }
        }
        }
        } else {
        if color(Nat.0, Nat.5) {
        if color(Nat.0, Nat.6) {
        if color(Nat.0, Nat.7) {
        if color(Nat.0, Nat.8) {
        color(Nat.0, Nat.1) = color(Nat.0, Nat.3)
        color(Nat.0, Nat.3) = color(Nat.0, Nat.5)
        color(Nat.0, Nat.5) = color(Nat.0, Nat.6)
        good_four_1356
        good_four(Nat.1, Nat.3, Nat.5, Nat.6)
        four_edges_from_zero_share_color_intro(color, Nat.1, Nat.3, Nat.5, Nat.6)
        four_edges_from_zero_share_color(color)
        } else {
        color(Nat.0, Nat.1) = color(Nat.0, Nat.3)
        color(Nat.0, Nat.3) = color(Nat.0, Nat.5)
        color(Nat.0, Nat.5) = color(Nat.0, Nat.6)
        good_four_1356
        good_four(Nat.1, Nat.3, Nat.5, Nat.6)
        four_edges_from_zero_share_color_intro(color, Nat.1, Nat.3, Nat.5, Nat.6)
        four_edges_from_zero_share_color(color)
        }
        } else {
        if color(Nat.0, Nat.8) {
        color(Nat.0, Nat.1) = color(Nat.0, Nat.3)
        color(Nat.0, Nat.3) = color(Nat.0, Nat.5)
        color(Nat.0, Nat.5) = color(Nat.0, Nat.6)
        good_four_1356
        good_four(Nat.1, Nat.3, Nat.5, Nat.6)
        four_edges_from_zero_share_color_intro(color, Nat.1, Nat.3, Nat.5, Nat.6)
        four_edges_from_zero_share_color(color)
        } else {
        color(Nat.0, Nat.1) = color(Nat.0, Nat.3)
        color(Nat.0, Nat.3) = color(Nat.0, Nat.5)
        color(Nat.0, Nat.5) = color(Nat.0, Nat.6)
        good_four_1356
        good_four(Nat.1, Nat.3, Nat.5, Nat.6)
        four_edges_from_zero_share_color_intro(color, Nat.1, Nat.3, Nat.5, Nat.6)
        four_edges_from_zero_share_color(color)
        }
        }
        } else {
        if color(Nat.0, Nat.7) {
        if color(Nat.0, Nat.8) {
        color(Nat.0, Nat.1) = color(Nat.0, Nat.3)
        color(Nat.0, Nat.3) = color(Nat.0, Nat.5)
        color(Nat.0, Nat.5) = color(Nat.0, Nat.7)
        good_four_1357
        good_four(Nat.1, Nat.3, Nat.5, Nat.7)
        four_edges_from_zero_share_color_intro(color, Nat.1, Nat.3, Nat.5, Nat.7)
        four_edges_from_zero_share_color(color)
        } else {
        color(Nat.0, Nat.1) = color(Nat.0, Nat.3)
        color(Nat.0, Nat.3) = color(Nat.0, Nat.5)
        color(Nat.0, Nat.5) = color(Nat.0, Nat.7)
        good_four_1357
        good_four(Nat.1, Nat.3, Nat.5, Nat.7)
        four_edges_from_zero_share_color_intro(color, Nat.1, Nat.3, Nat.5, Nat.7)
        four_edges_from_zero_share_color(color)
        }
        } else {
        if color(Nat.0, Nat.8) {
        color(Nat.0, Nat.1) = color(Nat.0, Nat.3)
        color(Nat.0, Nat.3) = color(Nat.0, Nat.5)
        color(Nat.0, Nat.5) = color(Nat.0, Nat.8)
        good_four_1358
        good_four(Nat.1, Nat.3, Nat.5, Nat.8)
        four_edges_from_zero_share_color_intro(color, Nat.1, Nat.3, Nat.5, Nat.8)
        four_edges_from_zero_share_color(color)
        } else {
        color(Nat.0, Nat.2) = color(Nat.0, Nat.4)
        color(Nat.0, Nat.4) = color(Nat.0, Nat.6)
        color(Nat.0, Nat.6) = color(Nat.0, Nat.7)
        good_four_2467
        good_four(Nat.2, Nat.4, Nat.6, Nat.7)
        four_edges_from_zero_share_color_intro(color, Nat.2, Nat.4, Nat.6, Nat.7)
        four_edges_from_zero_share_color(color)
        }
        }
        }
        } else {
        if color(Nat.0, Nat.6) {
        if color(Nat.0, Nat.7) {
        if color(Nat.0, Nat.8) {
        color(Nat.0, Nat.1) = color(Nat.0, Nat.3)
        color(Nat.0, Nat.3) = color(Nat.0, Nat.6)
        color(Nat.0, Nat.6) = color(Nat.0, Nat.7)
        good_four_1367
        good_four(Nat.1, Nat.3, Nat.6, Nat.7)
        four_edges_from_zero_share_color_intro(color, Nat.1, Nat.3, Nat.6, Nat.7)
        four_edges_from_zero_share_color(color)
        } else {
        color(Nat.0, Nat.1) = color(Nat.0, Nat.3)
        color(Nat.0, Nat.3) = color(Nat.0, Nat.6)
        color(Nat.0, Nat.6) = color(Nat.0, Nat.7)
        good_four_1367
        good_four(Nat.1, Nat.3, Nat.6, Nat.7)
        four_edges_from_zero_share_color_intro(color, Nat.1, Nat.3, Nat.6, Nat.7)
        four_edges_from_zero_share_color(color)
        }
        } else {
        if color(Nat.0, Nat.8) {
        color(Nat.0, Nat.1) = color(Nat.0, Nat.3)
        color(Nat.0, Nat.3) = color(Nat.0, Nat.6)
        color(Nat.0, Nat.6) = color(Nat.0, Nat.8)
        good_four_1368
        good_four(Nat.1, Nat.3, Nat.6, Nat.8)
        four_edges_from_zero_share_color_intro(color, Nat.1, Nat.3, Nat.6, Nat.8)
        four_edges_from_zero_share_color(color)
        } else {
        color(Nat.0, Nat.2) = color(Nat.0, Nat.4)
        color(Nat.0, Nat.4) = color(Nat.0, Nat.5)
        color(Nat.0, Nat.5) = color(Nat.0, Nat.7)
        good_four_2457
        good_four(Nat.2, Nat.4, Nat.5, Nat.7)
        four_edges_from_zero_share_color_intro(color, Nat.2, Nat.4, Nat.5, Nat.7)
        four_edges_from_zero_share_color(color)
        }
        }
        } else {
        if color(Nat.0, Nat.7) {
        if color(Nat.0, Nat.8) {
        color(Nat.0, Nat.1) = color(Nat.0, Nat.3)
        color(Nat.0, Nat.3) = color(Nat.0, Nat.7)
        color(Nat.0, Nat.7) = color(Nat.0, Nat.8)
        good_four_1378
        good_four(Nat.1, Nat.3, Nat.7, Nat.8)
        four_edges_from_zero_share_color_intro(color, Nat.1, Nat.3, Nat.7, Nat.8)
        four_edges_from_zero_share_color(color)
        } else {
        color(Nat.0, Nat.2) = color(Nat.0, Nat.4)
        color(Nat.0, Nat.4) = color(Nat.0, Nat.5)
        color(Nat.0, Nat.5) = color(Nat.0, Nat.6)
        good_four_2456
        good_four(Nat.2, Nat.4, Nat.5, Nat.6)
        four_edges_from_zero_share_color_intro(color, Nat.2, Nat.4, Nat.5, Nat.6)
        four_edges_from_zero_share_color(color)
        }
        } else {
        if color(Nat.0, Nat.8) {
        color(Nat.0, Nat.2) = color(Nat.0, Nat.4)
        color(Nat.0, Nat.4) = color(Nat.0, Nat.5)
        color(Nat.0, Nat.5) = color(Nat.0, Nat.6)
        good_four_2456
        good_four(Nat.2, Nat.4, Nat.5, Nat.6)
        four_edges_from_zero_share_color_intro(color, Nat.2, Nat.4, Nat.5, Nat.6)
        four_edges_from_zero_share_color(color)
        } else {
        color(Nat.0, Nat.2) = color(Nat.0, Nat.4)
        color(Nat.0, Nat.4) = color(Nat.0, Nat.5)
        color(Nat.0, Nat.5) = color(Nat.0, Nat.6)
        good_four_2456
        good_four(Nat.2, Nat.4, Nat.5, Nat.6)
        four_edges_from_zero_share_color_intro(color, Nat.2, Nat.4, Nat.5, Nat.6)
        four_edges_from_zero_share_color(color)
        }
        }
        }
        }
        }
        } else {
        if color(Nat.0, Nat.4) {
        if color(Nat.0, Nat.5) {
        if color(Nat.0, Nat.6) {
        if color(Nat.0, Nat.7) {
        if color(Nat.0, Nat.8) {
        color(Nat.0, Nat.1) = color(Nat.0, Nat.4)
        color(Nat.0, Nat.4) = color(Nat.0, Nat.5)
        color(Nat.0, Nat.5) = color(Nat.0, Nat.6)
        good_four_1456
        good_four(Nat.1, Nat.4, Nat.5, Nat.6)
        four_edges_from_zero_share_color_intro(color, Nat.1, Nat.4, Nat.5, Nat.6)
        four_edges_from_zero_share_color(color)
        } else {
        color(Nat.0, Nat.1) = color(Nat.0, Nat.4)
        color(Nat.0, Nat.4) = color(Nat.0, Nat.5)
        color(Nat.0, Nat.5) = color(Nat.0, Nat.6)
        good_four_1456
        good_four(Nat.1, Nat.4, Nat.5, Nat.6)
        four_edges_from_zero_share_color_intro(color, Nat.1, Nat.4, Nat.5, Nat.6)
        four_edges_from_zero_share_color(color)
        }
        } else {
        if color(Nat.0, Nat.8) {
        color(Nat.0, Nat.1) = color(Nat.0, Nat.4)
        color(Nat.0, Nat.4) = color(Nat.0, Nat.5)
        color(Nat.0, Nat.5) = color(Nat.0, Nat.6)
        good_four_1456
        good_four(Nat.1, Nat.4, Nat.5, Nat.6)
        four_edges_from_zero_share_color_intro(color, Nat.1, Nat.4, Nat.5, Nat.6)
        four_edges_from_zero_share_color(color)
        } else {
        color(Nat.0, Nat.1) = color(Nat.0, Nat.4)
        color(Nat.0, Nat.4) = color(Nat.0, Nat.5)
        color(Nat.0, Nat.5) = color(Nat.0, Nat.6)
        good_four_1456
        good_four(Nat.1, Nat.4, Nat.5, Nat.6)
        four_edges_from_zero_share_color_intro(color, Nat.1, Nat.4, Nat.5, Nat.6)
        four_edges_from_zero_share_color(color)
        }
        }
        } else {
        if color(Nat.0, Nat.7) {
        if color(Nat.0, Nat.8) {
        color(Nat.0, Nat.1) = color(Nat.0, Nat.4)
        color(Nat.0, Nat.4) = color(Nat.0, Nat.5)
        color(Nat.0, Nat.5) = color(Nat.0, Nat.7)
        good_four_1457
        good_four(Nat.1, Nat.4, Nat.5, Nat.7)
        four_edges_from_zero_share_color_intro(color, Nat.1, Nat.4, Nat.5, Nat.7)
        four_edges_from_zero_share_color(color)
        } else {
        color(Nat.0, Nat.1) = color(Nat.0, Nat.4)
        color(Nat.0, Nat.4) = color(Nat.0, Nat.5)
        color(Nat.0, Nat.5) = color(Nat.0, Nat.7)
        good_four_1457
        good_four(Nat.1, Nat.4, Nat.5, Nat.7)
        four_edges_from_zero_share_color_intro(color, Nat.1, Nat.4, Nat.5, Nat.7)
        four_edges_from_zero_share_color(color)
        }
        } else {
        if color(Nat.0, Nat.8) {
        color(Nat.0, Nat.1) = color(Nat.0, Nat.4)
        color(Nat.0, Nat.4) = color(Nat.0, Nat.5)
        color(Nat.0, Nat.5) = color(Nat.0, Nat.8)
        good_four_1458
        good_four(Nat.1, Nat.4, Nat.5, Nat.8)
        four_edges_from_zero_share_color_intro(color, Nat.1, Nat.4, Nat.5, Nat.8)
        four_edges_from_zero_share_color(color)
        } else {
        color(Nat.0, Nat.2) = color(Nat.0, Nat.3)
        color(Nat.0, Nat.3) = color(Nat.0, Nat.6)
        color(Nat.0, Nat.6) = color(Nat.0, Nat.7)
        good_four_2367
        good_four(Nat.2, Nat.3, Nat.6, Nat.7)
        four_edges_from_zero_share_color_intro(color, Nat.2, Nat.3, Nat.6, Nat.7)
        four_edges_from_zero_share_color(color)
        }
        }
        }
        } else {
        if color(Nat.0, Nat.6) {
        if color(Nat.0, Nat.7) {
        if color(Nat.0, Nat.8) {
        color(Nat.0, Nat.1) = color(Nat.0, Nat.4)
        color(Nat.0, Nat.4) = color(Nat.0, Nat.6)
        color(Nat.0, Nat.6) = color(Nat.0, Nat.7)
        good_four_1467
        good_four(Nat.1, Nat.4, Nat.6, Nat.7)
        four_edges_from_zero_share_color_intro(color, Nat.1, Nat.4, Nat.6, Nat.7)
        four_edges_from_zero_share_color(color)
        } else {
        color(Nat.0, Nat.1) = color(Nat.0, Nat.4)
        color(Nat.0, Nat.4) = color(Nat.0, Nat.6)
        color(Nat.0, Nat.6) = color(Nat.0, Nat.7)
        good_four_1467
        good_four(Nat.1, Nat.4, Nat.6, Nat.7)
        four_edges_from_zero_share_color_intro(color, Nat.1, Nat.4, Nat.6, Nat.7)
        four_edges_from_zero_share_color(color)
        }
        } else {
        if color(Nat.0, Nat.8) {
        color(Nat.0, Nat.1) = color(Nat.0, Nat.4)
        color(Nat.0, Nat.4) = color(Nat.0, Nat.6)
        color(Nat.0, Nat.6) = color(Nat.0, Nat.8)
        good_four_1468
        good_four(Nat.1, Nat.4, Nat.6, Nat.8)
        four_edges_from_zero_share_color_intro(color, Nat.1, Nat.4, Nat.6, Nat.8)
        four_edges_from_zero_share_color(color)
        } else {
        color(Nat.0, Nat.2) = color(Nat.0, Nat.3)
        color(Nat.0, Nat.3) = color(Nat.0, Nat.5)
        color(Nat.0, Nat.5) = color(Nat.0, Nat.7)
        good_four_2357
        good_four(Nat.2, Nat.3, Nat.5, Nat.7)
        four_edges_from_zero_share_color_intro(color, Nat.2, Nat.3, Nat.5, Nat.7)
        four_edges_from_zero_share_color(color)
        }
        }
        } else {
        if color(Nat.0, Nat.7) {
        if color(Nat.0, Nat.8) {
        color(Nat.0, Nat.1) = color(Nat.0, Nat.4)
        color(Nat.0, Nat.4) = color(Nat.0, Nat.7)
        color(Nat.0, Nat.7) = color(Nat.0, Nat.8)
        good_four_1478
        good_four(Nat.1, Nat.4, Nat.7, Nat.8)
        four_edges_from_zero_share_color_intro(color, Nat.1, Nat.4, Nat.7, Nat.8)
        four_edges_from_zero_share_color(color)
        } else {
        color(Nat.0, Nat.2) = color(Nat.0, Nat.3)
        color(Nat.0, Nat.3) = color(Nat.0, Nat.5)
        color(Nat.0, Nat.5) = color(Nat.0, Nat.6)
        good_four_2356
        good_four(Nat.2, Nat.3, Nat.5, Nat.6)
        four_edges_from_zero_share_color_intro(color, Nat.2, Nat.3, Nat.5, Nat.6)
        four_edges_from_zero_share_color(color)
        }
        } else {
        if color(Nat.0, Nat.8) {
        color(Nat.0, Nat.2) = color(Nat.0, Nat.3)
        color(Nat.0, Nat.3) = color(Nat.0, Nat.5)
        color(Nat.0, Nat.5) = color(Nat.0, Nat.6)
        good_four_2356
        good_four(Nat.2, Nat.3, Nat.5, Nat.6)
        four_edges_from_zero_share_color_intro(color, Nat.2, Nat.3, Nat.5, Nat.6)
        four_edges_from_zero_share_color(color)
        } else {
        color(Nat.0, Nat.2) = color(Nat.0, Nat.3)
        color(Nat.0, Nat.3) = color(Nat.0, Nat.5)
        color(Nat.0, Nat.5) = color(Nat.0, Nat.6)
        good_four_2356
        good_four(Nat.2, Nat.3, Nat.5, Nat.6)
        four_edges_from_zero_share_color_intro(color, Nat.2, Nat.3, Nat.5, Nat.6)
        four_edges_from_zero_share_color(color)
        }
        }
        }
        }
        } else {
        if color(Nat.0, Nat.5) {
        if color(Nat.0, Nat.6) {
        if color(Nat.0, Nat.7) {
        if color(Nat.0, Nat.8) {
        color(Nat.0, Nat.1) = color(Nat.0, Nat.5)
        color(Nat.0, Nat.5) = color(Nat.0, Nat.6)
        color(Nat.0, Nat.6) = color(Nat.0, Nat.7)
        good_four_1567
        good_four(Nat.1, Nat.5, Nat.6, Nat.7)
        four_edges_from_zero_share_color_intro(color, Nat.1, Nat.5, Nat.6, Nat.7)
        four_edges_from_zero_share_color(color)
        } else {
        color(Nat.0, Nat.1) = color(Nat.0, Nat.5)
        color(Nat.0, Nat.5) = color(Nat.0, Nat.6)
        color(Nat.0, Nat.6) = color(Nat.0, Nat.7)
        good_four_1567
        good_four(Nat.1, Nat.5, Nat.6, Nat.7)
        four_edges_from_zero_share_color_intro(color, Nat.1, Nat.5, Nat.6, Nat.7)
        four_edges_from_zero_share_color(color)
        }
        } else {
        if color(Nat.0, Nat.8) {
        color(Nat.0, Nat.1) = color(Nat.0, Nat.5)
        color(Nat.0, Nat.5) = color(Nat.0, Nat.6)
        color(Nat.0, Nat.6) = color(Nat.0, Nat.8)
        good_four_1568
        good_four(Nat.1, Nat.5, Nat.6, Nat.8)
        four_edges_from_zero_share_color_intro(color, Nat.1, Nat.5, Nat.6, Nat.8)
        four_edges_from_zero_share_color(color)
        } else {
        color(Nat.0, Nat.2) = color(Nat.0, Nat.3)
        color(Nat.0, Nat.3) = color(Nat.0, Nat.4)
        color(Nat.0, Nat.4) = color(Nat.0, Nat.7)
        good_four_2347
        good_four(Nat.2, Nat.3, Nat.4, Nat.7)
        four_edges_from_zero_share_color_intro(color, Nat.2, Nat.3, Nat.4, Nat.7)
        four_edges_from_zero_share_color(color)
        }
        }
        } else {
        if color(Nat.0, Nat.7) {
        if color(Nat.0, Nat.8) {
        color(Nat.0, Nat.1) = color(Nat.0, Nat.5)
        color(Nat.0, Nat.5) = color(Nat.0, Nat.7)
        color(Nat.0, Nat.7) = color(Nat.0, Nat.8)
        good_four_1578
        good_four(Nat.1, Nat.5, Nat.7, Nat.8)
        four_edges_from_zero_share_color_intro(color, Nat.1, Nat.5, Nat.7, Nat.8)
        four_edges_from_zero_share_color(color)
        } else {
        color(Nat.0, Nat.2) = color(Nat.0, Nat.3)
        color(Nat.0, Nat.3) = color(Nat.0, Nat.4)
        color(Nat.0, Nat.4) = color(Nat.0, Nat.6)
        good_four_2346
        good_four(Nat.2, Nat.3, Nat.4, Nat.6)
        four_edges_from_zero_share_color_intro(color, Nat.2, Nat.3, Nat.4, Nat.6)
        four_edges_from_zero_share_color(color)
        }
        } else {
        if color(Nat.0, Nat.8) {
        color(Nat.0, Nat.2) = color(Nat.0, Nat.3)
        color(Nat.0, Nat.3) = color(Nat.0, Nat.4)
        color(Nat.0, Nat.4) = color(Nat.0, Nat.6)
        good_four_2346
        good_four(Nat.2, Nat.3, Nat.4, Nat.6)
        four_edges_from_zero_share_color_intro(color, Nat.2, Nat.3, Nat.4, Nat.6)
        four_edges_from_zero_share_color(color)
        } else {
        color(Nat.0, Nat.2) = color(Nat.0, Nat.3)
        color(Nat.0, Nat.3) = color(Nat.0, Nat.4)
        color(Nat.0, Nat.4) = color(Nat.0, Nat.6)
        good_four_2346
        good_four(Nat.2, Nat.3, Nat.4, Nat.6)
        four_edges_from_zero_share_color_intro(color, Nat.2, Nat.3, Nat.4, Nat.6)
        four_edges_from_zero_share_color(color)
        }
        }
        }
        } else {
        if color(Nat.0, Nat.6) {
        if color(Nat.0, Nat.7) {
        if color(Nat.0, Nat.8) {
        color(Nat.0, Nat.1) = color(Nat.0, Nat.6)
        color(Nat.0, Nat.6) = color(Nat.0, Nat.7)
        color(Nat.0, Nat.7) = color(Nat.0, Nat.8)
        good_four_1678
        good_four(Nat.1, Nat.6, Nat.7, Nat.8)
        four_edges_from_zero_share_color_intro(color, Nat.1, Nat.6, Nat.7, Nat.8)
        four_edges_from_zero_share_color(color)
        } else {
        color(Nat.0, Nat.2) = color(Nat.0, Nat.3)
        color(Nat.0, Nat.3) = color(Nat.0, Nat.4)
        color(Nat.0, Nat.4) = color(Nat.0, Nat.5)
        good_four_2345
        good_four(Nat.2, Nat.3, Nat.4, Nat.5)
        four_edges_from_zero_share_color_intro(color, Nat.2, Nat.3, Nat.4, Nat.5)
        four_edges_from_zero_share_color(color)
        }
        } else {
        if color(Nat.0, Nat.8) {
        color(Nat.0, Nat.2) = color(Nat.0, Nat.3)
        color(Nat.0, Nat.3) = color(Nat.0, Nat.4)
        color(Nat.0, Nat.4) = color(Nat.0, Nat.5)
        good_four_2345
        good_four(Nat.2, Nat.3, Nat.4, Nat.5)
        four_edges_from_zero_share_color_intro(color, Nat.2, Nat.3, Nat.4, Nat.5)
        four_edges_from_zero_share_color(color)
        } else {
        color(Nat.0, Nat.2) = color(Nat.0, Nat.3)
        color(Nat.0, Nat.3) = color(Nat.0, Nat.4)
        color(Nat.0, Nat.4) = color(Nat.0, Nat.5)
        good_four_2345
        good_four(Nat.2, Nat.3, Nat.4, Nat.5)
        four_edges_from_zero_share_color_intro(color, Nat.2, Nat.3, Nat.4, Nat.5)
        four_edges_from_zero_share_color(color)
        }
        }
        } else {
        if color(Nat.0, Nat.7) {
        if color(Nat.0, Nat.8) {
        color(Nat.0, Nat.2) = color(Nat.0, Nat.3)
        color(Nat.0, Nat.3) = color(Nat.0, Nat.4)
        color(Nat.0, Nat.4) = color(Nat.0, Nat.5)
        good_four_2345
        good_four(Nat.2, Nat.3, Nat.4, Nat.5)
        four_edges_from_zero_share_color_intro(color, Nat.2, Nat.3, Nat.4, Nat.5)
        four_edges_from_zero_share_color(color)
        } else {
        color(Nat.0, Nat.2) = color(Nat.0, Nat.3)
        color(Nat.0, Nat.3) = color(Nat.0, Nat.4)
        color(Nat.0, Nat.4) = color(Nat.0, Nat.5)
        good_four_2345
        good_four(Nat.2, Nat.3, Nat.4, Nat.5)
        four_edges_from_zero_share_color_intro(color, Nat.2, Nat.3, Nat.4, Nat.5)
        four_edges_from_zero_share_color(color)
        }
        } else {
        if color(Nat.0, Nat.8) {
        color(Nat.0, Nat.2) = color(Nat.0, Nat.3)
        color(Nat.0, Nat.3) = color(Nat.0, Nat.4)
        color(Nat.0, Nat.4) = color(Nat.0, Nat.5)
        good_four_2345
        good_four(Nat.2, Nat.3, Nat.4, Nat.5)
        four_edges_from_zero_share_color_intro(color, Nat.2, Nat.3, Nat.4, Nat.5)
        four_edges_from_zero_share_color(color)
        } else {
        color(Nat.0, Nat.2) = color(Nat.0, Nat.3)
        color(Nat.0, Nat.3) = color(Nat.0, Nat.4)
        color(Nat.0, Nat.4) = color(Nat.0, Nat.5)
        good_four_2345
        good_four(Nat.2, Nat.3, Nat.4, Nat.5)
        four_edges_from_zero_share_color_intro(color, Nat.2, Nat.3, Nat.4, Nat.5)
        four_edges_from_zero_share_color(color)
        }
        }
        }
        }
        }
        }
        }
        } else {
        if color(Nat.0, Nat.2) {
        if color(Nat.0, Nat.3) {
        if color(Nat.0, Nat.4) {
        if color(Nat.0, Nat.5) {
        if color(Nat.0, Nat.6) {
        if color(Nat.0, Nat.7) {
        if color(Nat.0, Nat.8) {
        color(Nat.0, Nat.2) = color(Nat.0, Nat.3)
        color(Nat.0, Nat.3) = color(Nat.0, Nat.4)
        color(Nat.0, Nat.4) = color(Nat.0, Nat.5)
        good_four_2345
        good_four(Nat.2, Nat.3, Nat.4, Nat.5)
        four_edges_from_zero_share_color_intro(color, Nat.2, Nat.3, Nat.4, Nat.5)
        four_edges_from_zero_share_color(color)
        } else {
        color(Nat.0, Nat.2) = color(Nat.0, Nat.3)
        color(Nat.0, Nat.3) = color(Nat.0, Nat.4)
        color(Nat.0, Nat.4) = color(Nat.0, Nat.5)
        good_four_2345
        good_four(Nat.2, Nat.3, Nat.4, Nat.5)
        four_edges_from_zero_share_color_intro(color, Nat.2, Nat.3, Nat.4, Nat.5)
        four_edges_from_zero_share_color(color)
        }
        } else {
        if color(Nat.0, Nat.8) {
        color(Nat.0, Nat.2) = color(Nat.0, Nat.3)
        color(Nat.0, Nat.3) = color(Nat.0, Nat.4)
        color(Nat.0, Nat.4) = color(Nat.0, Nat.5)
        good_four_2345
        good_four(Nat.2, Nat.3, Nat.4, Nat.5)
        four_edges_from_zero_share_color_intro(color, Nat.2, Nat.3, Nat.4, Nat.5)
        four_edges_from_zero_share_color(color)
        } else {
        color(Nat.0, Nat.2) = color(Nat.0, Nat.3)
        color(Nat.0, Nat.3) = color(Nat.0, Nat.4)
        color(Nat.0, Nat.4) = color(Nat.0, Nat.5)
        good_four_2345
        good_four(Nat.2, Nat.3, Nat.4, Nat.5)
        four_edges_from_zero_share_color_intro(color, Nat.2, Nat.3, Nat.4, Nat.5)
        four_edges_from_zero_share_color(color)
        }
        }
        } else {
        if color(Nat.0, Nat.7) {
        if color(Nat.0, Nat.8) {
        color(Nat.0, Nat.2) = color(Nat.0, Nat.3)
        color(Nat.0, Nat.3) = color(Nat.0, Nat.4)
        color(Nat.0, Nat.4) = color(Nat.0, Nat.5)
        good_four_2345
        good_four(Nat.2, Nat.3, Nat.4, Nat.5)
        four_edges_from_zero_share_color_intro(color, Nat.2, Nat.3, Nat.4, Nat.5)
        four_edges_from_zero_share_color(color)
        } else {
        color(Nat.0, Nat.2) = color(Nat.0, Nat.3)
        color(Nat.0, Nat.3) = color(Nat.0, Nat.4)
        color(Nat.0, Nat.4) = color(Nat.0, Nat.5)
        good_four_2345
        good_four(Nat.2, Nat.3, Nat.4, Nat.5)
        four_edges_from_zero_share_color_intro(color, Nat.2, Nat.3, Nat.4, Nat.5)
        four_edges_from_zero_share_color(color)
        }
        } else {
        if color(Nat.0, Nat.8) {
        color(Nat.0, Nat.2) = color(Nat.0, Nat.3)
        color(Nat.0, Nat.3) = color(Nat.0, Nat.4)
        color(Nat.0, Nat.4) = color(Nat.0, Nat.5)
        good_four_2345
        good_four(Nat.2, Nat.3, Nat.4, Nat.5)
        four_edges_from_zero_share_color_intro(color, Nat.2, Nat.3, Nat.4, Nat.5)
        four_edges_from_zero_share_color(color)
        } else {
        color(Nat.0, Nat.2) = color(Nat.0, Nat.3)
        color(Nat.0, Nat.3) = color(Nat.0, Nat.4)
        color(Nat.0, Nat.4) = color(Nat.0, Nat.5)
        good_four_2345
        good_four(Nat.2, Nat.3, Nat.4, Nat.5)
        four_edges_from_zero_share_color_intro(color, Nat.2, Nat.3, Nat.4, Nat.5)
        four_edges_from_zero_share_color(color)
        }
        }
        }
        } else {
        if color(Nat.0, Nat.6) {
        if color(Nat.0, Nat.7) {
        if color(Nat.0, Nat.8) {
        color(Nat.0, Nat.2) = color(Nat.0, Nat.3)
        color(Nat.0, Nat.3) = color(Nat.0, Nat.4)
        color(Nat.0, Nat.4) = color(Nat.0, Nat.6)
        good_four_2346
        good_four(Nat.2, Nat.3, Nat.4, Nat.6)
        four_edges_from_zero_share_color_intro(color, Nat.2, Nat.3, Nat.4, Nat.6)
        four_edges_from_zero_share_color(color)
        } else {
        color(Nat.0, Nat.2) = color(Nat.0, Nat.3)
        color(Nat.0, Nat.3) = color(Nat.0, Nat.4)
        color(Nat.0, Nat.4) = color(Nat.0, Nat.6)
        good_four_2346
        good_four(Nat.2, Nat.3, Nat.4, Nat.6)
        four_edges_from_zero_share_color_intro(color, Nat.2, Nat.3, Nat.4, Nat.6)
        four_edges_from_zero_share_color(color)
        }
        } else {
        if color(Nat.0, Nat.8) {
        color(Nat.0, Nat.2) = color(Nat.0, Nat.3)
        color(Nat.0, Nat.3) = color(Nat.0, Nat.4)
        color(Nat.0, Nat.4) = color(Nat.0, Nat.6)
        good_four_2346
        good_four(Nat.2, Nat.3, Nat.4, Nat.6)
        four_edges_from_zero_share_color_intro(color, Nat.2, Nat.3, Nat.4, Nat.6)
        four_edges_from_zero_share_color(color)
        } else {
        color(Nat.0, Nat.2) = color(Nat.0, Nat.3)
        color(Nat.0, Nat.3) = color(Nat.0, Nat.4)
        color(Nat.0, Nat.4) = color(Nat.0, Nat.6)
        good_four_2346
        good_four(Nat.2, Nat.3, Nat.4, Nat.6)
        four_edges_from_zero_share_color_intro(color, Nat.2, Nat.3, Nat.4, Nat.6)
        four_edges_from_zero_share_color(color)
        }
        }
        } else {
        if color(Nat.0, Nat.7) {
        if color(Nat.0, Nat.8) {
        color(Nat.0, Nat.2) = color(Nat.0, Nat.3)
        color(Nat.0, Nat.3) = color(Nat.0, Nat.4)
        color(Nat.0, Nat.4) = color(Nat.0, Nat.7)
        good_four_2347
        good_four(Nat.2, Nat.3, Nat.4, Nat.7)
        four_edges_from_zero_share_color_intro(color, Nat.2, Nat.3, Nat.4, Nat.7)
        four_edges_from_zero_share_color(color)
        } else {
        color(Nat.0, Nat.2) = color(Nat.0, Nat.3)
        color(Nat.0, Nat.3) = color(Nat.0, Nat.4)
        color(Nat.0, Nat.4) = color(Nat.0, Nat.7)
        good_four_2347
        good_four(Nat.2, Nat.3, Nat.4, Nat.7)
        four_edges_from_zero_share_color_intro(color, Nat.2, Nat.3, Nat.4, Nat.7)
        four_edges_from_zero_share_color(color)
        }
        } else {
        if color(Nat.0, Nat.8) {
        color(Nat.0, Nat.2) = color(Nat.0, Nat.3)
        color(Nat.0, Nat.3) = color(Nat.0, Nat.4)
        color(Nat.0, Nat.4) = color(Nat.0, Nat.8)
        good_four_2348
        good_four(Nat.2, Nat.3, Nat.4, Nat.8)
        four_edges_from_zero_share_color_intro(color, Nat.2, Nat.3, Nat.4, Nat.8)
        four_edges_from_zero_share_color(color)
        } else {
        color(Nat.0, Nat.1) = color(Nat.0, Nat.5)
        color(Nat.0, Nat.5) = color(Nat.0, Nat.6)
        color(Nat.0, Nat.6) = color(Nat.0, Nat.7)
        good_four_1567
        good_four(Nat.1, Nat.5, Nat.6, Nat.7)
        four_edges_from_zero_share_color_intro(color, Nat.1, Nat.5, Nat.6, Nat.7)
        four_edges_from_zero_share_color(color)
        }
        }
        }
        }
        } else {
        if color(Nat.0, Nat.5) {
        if color(Nat.0, Nat.6) {
        if color(Nat.0, Nat.7) {
        if color(Nat.0, Nat.8) {
        color(Nat.0, Nat.2) = color(Nat.0, Nat.3)
        color(Nat.0, Nat.3) = color(Nat.0, Nat.5)
        color(Nat.0, Nat.5) = color(Nat.0, Nat.6)
        good_four_2356
        good_four(Nat.2, Nat.3, Nat.5, Nat.6)
        four_edges_from_zero_share_color_intro(color, Nat.2, Nat.3, Nat.5, Nat.6)
        four_edges_from_zero_share_color(color)
        } else {
        color(Nat.0, Nat.2) = color(Nat.0, Nat.3)
        color(Nat.0, Nat.3) = color(Nat.0, Nat.5)
        color(Nat.0, Nat.5) = color(Nat.0, Nat.6)
        good_four_2356
        good_four(Nat.2, Nat.3, Nat.5, Nat.6)
        four_edges_from_zero_share_color_intro(color, Nat.2, Nat.3, Nat.5, Nat.6)
        four_edges_from_zero_share_color(color)
        }
        } else {
        if color(Nat.0, Nat.8) {
        color(Nat.0, Nat.2) = color(Nat.0, Nat.3)
        color(Nat.0, Nat.3) = color(Nat.0, Nat.5)
        color(Nat.0, Nat.5) = color(Nat.0, Nat.6)
        good_four_2356
        good_four(Nat.2, Nat.3, Nat.5, Nat.6)
        four_edges_from_zero_share_color_intro(color, Nat.2, Nat.3, Nat.5, Nat.6)
        four_edges_from_zero_share_color(color)
        } else {
        color(Nat.0, Nat.2) = color(Nat.0, Nat.3)
        color(Nat.0, Nat.3) = color(Nat.0, Nat.5)
        color(Nat.0, Nat.5) = color(Nat.0, Nat.6)
        good_four_2356
        good_four(Nat.2, Nat.3, Nat.5, Nat.6)
        four_edges_from_zero_share_color_intro(color, Nat.2, Nat.3, Nat.5, Nat.6)
        four_edges_from_zero_share_color(color)
        }
        }
        } else {
        if color(Nat.0, Nat.7) {
        if color(Nat.0, Nat.8) {
        color(Nat.0, Nat.2) = color(Nat.0, Nat.3)
        color(Nat.0, Nat.3) = color(Nat.0, Nat.5)
        color(Nat.0, Nat.5) = color(Nat.0, Nat.7)
        good_four_2357
        good_four(Nat.2, Nat.3, Nat.5, Nat.7)
        four_edges_from_zero_share_color_intro(color, Nat.2, Nat.3, Nat.5, Nat.7)
        four_edges_from_zero_share_color(color)
        } else {
        color(Nat.0, Nat.2) = color(Nat.0, Nat.3)
        color(Nat.0, Nat.3) = color(Nat.0, Nat.5)
        color(Nat.0, Nat.5) = color(Nat.0, Nat.7)
        good_four_2357
        good_four(Nat.2, Nat.3, Nat.5, Nat.7)
        four_edges_from_zero_share_color_intro(color, Nat.2, Nat.3, Nat.5, Nat.7)
        four_edges_from_zero_share_color(color)
        }
        } else {
        if color(Nat.0, Nat.8) {
        color(Nat.0, Nat.2) = color(Nat.0, Nat.3)
        color(Nat.0, Nat.3) = color(Nat.0, Nat.5)
        color(Nat.0, Nat.5) = color(Nat.0, Nat.8)
        good_four_2358
        good_four(Nat.2, Nat.3, Nat.5, Nat.8)
        four_edges_from_zero_share_color_intro(color, Nat.2, Nat.3, Nat.5, Nat.8)
        four_edges_from_zero_share_color(color)
        } else {
        color(Nat.0, Nat.1) = color(Nat.0, Nat.4)
        color(Nat.0, Nat.4) = color(Nat.0, Nat.6)
        color(Nat.0, Nat.6) = color(Nat.0, Nat.7)
        good_four_1467
        good_four(Nat.1, Nat.4, Nat.6, Nat.7)
        four_edges_from_zero_share_color_intro(color, Nat.1, Nat.4, Nat.6, Nat.7)
        four_edges_from_zero_share_color(color)
        }
        }
        }
        } else {
        if color(Nat.0, Nat.6) {
        if color(Nat.0, Nat.7) {
        if color(Nat.0, Nat.8) {
        color(Nat.0, Nat.2) = color(Nat.0, Nat.3)
        color(Nat.0, Nat.3) = color(Nat.0, Nat.6)
        color(Nat.0, Nat.6) = color(Nat.0, Nat.7)
        good_four_2367
        good_four(Nat.2, Nat.3, Nat.6, Nat.7)
        four_edges_from_zero_share_color_intro(color, Nat.2, Nat.3, Nat.6, Nat.7)
        four_edges_from_zero_share_color(color)
        } else {
        color(Nat.0, Nat.2) = color(Nat.0, Nat.3)
        color(Nat.0, Nat.3) = color(Nat.0, Nat.6)
        color(Nat.0, Nat.6) = color(Nat.0, Nat.7)
        good_four_2367
        good_four(Nat.2, Nat.3, Nat.6, Nat.7)
        four_edges_from_zero_share_color_intro(color, Nat.2, Nat.3, Nat.6, Nat.7)
        four_edges_from_zero_share_color(color)
        }
        } else {
        if color(Nat.0, Nat.8) {
        color(Nat.0, Nat.2) = color(Nat.0, Nat.3)
        color(Nat.0, Nat.3) = color(Nat.0, Nat.6)
        color(Nat.0, Nat.6) = color(Nat.0, Nat.8)
        good_four_2368
        good_four(Nat.2, Nat.3, Nat.6, Nat.8)
        four_edges_from_zero_share_color_intro(color, Nat.2, Nat.3, Nat.6, Nat.8)
        four_edges_from_zero_share_color(color)
        } else {
        color(Nat.0, Nat.1) = color(Nat.0, Nat.4)
        color(Nat.0, Nat.4) = color(Nat.0, Nat.5)
        color(Nat.0, Nat.5) = color(Nat.0, Nat.7)
        good_four_1457
        good_four(Nat.1, Nat.4, Nat.5, Nat.7)
        four_edges_from_zero_share_color_intro(color, Nat.1, Nat.4, Nat.5, Nat.7)
        four_edges_from_zero_share_color(color)
        }
        }
        } else {
        if color(Nat.0, Nat.7) {
        if color(Nat.0, Nat.8) {
        color(Nat.0, Nat.2) = color(Nat.0, Nat.3)
        color(Nat.0, Nat.3) = color(Nat.0, Nat.7)
        color(Nat.0, Nat.7) = color(Nat.0, Nat.8)
        good_four_2378
        good_four(Nat.2, Nat.3, Nat.7, Nat.8)
        four_edges_from_zero_share_color_intro(color, Nat.2, Nat.3, Nat.7, Nat.8)
        four_edges_from_zero_share_color(color)
        } else {
        color(Nat.0, Nat.1) = color(Nat.0, Nat.4)
        color(Nat.0, Nat.4) = color(Nat.0, Nat.5)
        color(Nat.0, Nat.5) = color(Nat.0, Nat.6)
        good_four_1456
        good_four(Nat.1, Nat.4, Nat.5, Nat.6)
        four_edges_from_zero_share_color_intro(color, Nat.1, Nat.4, Nat.5, Nat.6)
        four_edges_from_zero_share_color(color)
        }
        } else {
        if color(Nat.0, Nat.8) {
        color(Nat.0, Nat.1) = color(Nat.0, Nat.4)
        color(Nat.0, Nat.4) = color(Nat.0, Nat.5)
        color(Nat.0, Nat.5) = color(Nat.0, Nat.6)
        good_four_1456
        good_four(Nat.1, Nat.4, Nat.5, Nat.6)
        four_edges_from_zero_share_color_intro(color, Nat.1, Nat.4, Nat.5, Nat.6)
        four_edges_from_zero_share_color(color)
        } else {
        color(Nat.0, Nat.1) = color(Nat.0, Nat.4)
        color(Nat.0, Nat.4) = color(Nat.0, Nat.5)
        color(Nat.0, Nat.5) = color(Nat.0, Nat.6)
        good_four_1456
        good_four(Nat.1, Nat.4, Nat.5, Nat.6)
        four_edges_from_zero_share_color_intro(color, Nat.1, Nat.4, Nat.5, Nat.6)
        four_edges_from_zero_share_color(color)
        }
        }
        }
        }
        }
        } else {
        if color(Nat.0, Nat.4) {
        if color(Nat.0, Nat.5) {
        if color(Nat.0, Nat.6) {
        if color(Nat.0, Nat.7) {
        if color(Nat.0, Nat.8) {
        color(Nat.0, Nat.2) = color(Nat.0, Nat.4)
        color(Nat.0, Nat.4) = color(Nat.0, Nat.5)
        color(Nat.0, Nat.5) = color(Nat.0, Nat.6)
        good_four_2456
        good_four(Nat.2, Nat.4, Nat.5, Nat.6)
        four_edges_from_zero_share_color_intro(color, Nat.2, Nat.4, Nat.5, Nat.6)
        four_edges_from_zero_share_color(color)
        } else {
        color(Nat.0, Nat.2) = color(Nat.0, Nat.4)
        color(Nat.0, Nat.4) = color(Nat.0, Nat.5)
        color(Nat.0, Nat.5) = color(Nat.0, Nat.6)
        good_four_2456
        good_four(Nat.2, Nat.4, Nat.5, Nat.6)
        four_edges_from_zero_share_color_intro(color, Nat.2, Nat.4, Nat.5, Nat.6)
        four_edges_from_zero_share_color(color)
        }
        } else {
        if color(Nat.0, Nat.8) {
        color(Nat.0, Nat.2) = color(Nat.0, Nat.4)
        color(Nat.0, Nat.4) = color(Nat.0, Nat.5)
        color(Nat.0, Nat.5) = color(Nat.0, Nat.6)
        good_four_2456
        good_four(Nat.2, Nat.4, Nat.5, Nat.6)
        four_edges_from_zero_share_color_intro(color, Nat.2, Nat.4, Nat.5, Nat.6)
        four_edges_from_zero_share_color(color)
        } else {
        color(Nat.0, Nat.2) = color(Nat.0, Nat.4)
        color(Nat.0, Nat.4) = color(Nat.0, Nat.5)
        color(Nat.0, Nat.5) = color(Nat.0, Nat.6)
        good_four_2456
        good_four(Nat.2, Nat.4, Nat.5, Nat.6)
        four_edges_from_zero_share_color_intro(color, Nat.2, Nat.4, Nat.5, Nat.6)
        four_edges_from_zero_share_color(color)
        }
        }
        } else {
        if color(Nat.0, Nat.7) {
        if color(Nat.0, Nat.8) {
        color(Nat.0, Nat.2) = color(Nat.0, Nat.4)
        color(Nat.0, Nat.4) = color(Nat.0, Nat.5)
        color(Nat.0, Nat.5) = color(Nat.0, Nat.7)
        good_four_2457
        good_four(Nat.2, Nat.4, Nat.5, Nat.7)
        four_edges_from_zero_share_color_intro(color, Nat.2, Nat.4, Nat.5, Nat.7)
        four_edges_from_zero_share_color(color)
        } else {
        color(Nat.0, Nat.2) = color(Nat.0, Nat.4)
        color(Nat.0, Nat.4) = color(Nat.0, Nat.5)
        color(Nat.0, Nat.5) = color(Nat.0, Nat.7)
        good_four_2457
        good_four(Nat.2, Nat.4, Nat.5, Nat.7)
        four_edges_from_zero_share_color_intro(color, Nat.2, Nat.4, Nat.5, Nat.7)
        four_edges_from_zero_share_color(color)
        }
        } else {
        if color(Nat.0, Nat.8) {
        color(Nat.0, Nat.2) = color(Nat.0, Nat.4)
        color(Nat.0, Nat.4) = color(Nat.0, Nat.5)
        color(Nat.0, Nat.5) = color(Nat.0, Nat.8)
        good_four_2458
        good_four(Nat.2, Nat.4, Nat.5, Nat.8)
        four_edges_from_zero_share_color_intro(color, Nat.2, Nat.4, Nat.5, Nat.8)
        four_edges_from_zero_share_color(color)
        } else {
        color(Nat.0, Nat.1) = color(Nat.0, Nat.3)
        color(Nat.0, Nat.3) = color(Nat.0, Nat.6)
        color(Nat.0, Nat.6) = color(Nat.0, Nat.7)
        good_four_1367
        good_four(Nat.1, Nat.3, Nat.6, Nat.7)
        four_edges_from_zero_share_color_intro(color, Nat.1, Nat.3, Nat.6, Nat.7)
        four_edges_from_zero_share_color(color)
        }
        }
        }
        } else {
        if color(Nat.0, Nat.6) {
        if color(Nat.0, Nat.7) {
        if color(Nat.0, Nat.8) {
        color(Nat.0, Nat.2) = color(Nat.0, Nat.4)
        color(Nat.0, Nat.4) = color(Nat.0, Nat.6)
        color(Nat.0, Nat.6) = color(Nat.0, Nat.7)
        good_four_2467
        good_four(Nat.2, Nat.4, Nat.6, Nat.7)
        four_edges_from_zero_share_color_intro(color, Nat.2, Nat.4, Nat.6, Nat.7)
        four_edges_from_zero_share_color(color)
        } else {
        color(Nat.0, Nat.2) = color(Nat.0, Nat.4)
        color(Nat.0, Nat.4) = color(Nat.0, Nat.6)
        color(Nat.0, Nat.6) = color(Nat.0, Nat.7)
        good_four_2467
        good_four(Nat.2, Nat.4, Nat.6, Nat.7)
        four_edges_from_zero_share_color_intro(color, Nat.2, Nat.4, Nat.6, Nat.7)
        four_edges_from_zero_share_color(color)
        }
        } else {
        if color(Nat.0, Nat.8) {
        color(Nat.0, Nat.2) = color(Nat.0, Nat.4)
        color(Nat.0, Nat.4) = color(Nat.0, Nat.6)
        color(Nat.0, Nat.6) = color(Nat.0, Nat.8)
        good_four_2468
        good_four(Nat.2, Nat.4, Nat.6, Nat.8)
        four_edges_from_zero_share_color_intro(color, Nat.2, Nat.4, Nat.6, Nat.8)
        four_edges_from_zero_share_color(color)
        } else {
        color(Nat.0, Nat.1) = color(Nat.0, Nat.3)
        color(Nat.0, Nat.3) = color(Nat.0, Nat.5)
        color(Nat.0, Nat.5) = color(Nat.0, Nat.7)
        good_four_1357
        good_four(Nat.1, Nat.3, Nat.5, Nat.7)
        four_edges_from_zero_share_color_intro(color, Nat.1, Nat.3, Nat.5, Nat.7)
        four_edges_from_zero_share_color(color)
        }
        }
        } else {
        if color(Nat.0, Nat.7) {
        if color(Nat.0, Nat.8) {
        color(Nat.0, Nat.2) = color(Nat.0, Nat.4)
        color(Nat.0, Nat.4) = color(Nat.0, Nat.7)
        color(Nat.0, Nat.7) = color(Nat.0, Nat.8)
        good_four_2478
        good_four(Nat.2, Nat.4, Nat.7, Nat.8)
        four_edges_from_zero_share_color_intro(color, Nat.2, Nat.4, Nat.7, Nat.8)
        four_edges_from_zero_share_color(color)
        } else {
        color(Nat.0, Nat.1) = color(Nat.0, Nat.3)
        color(Nat.0, Nat.3) = color(Nat.0, Nat.5)
        color(Nat.0, Nat.5) = color(Nat.0, Nat.6)
        good_four_1356
        good_four(Nat.1, Nat.3, Nat.5, Nat.6)
        four_edges_from_zero_share_color_intro(color, Nat.1, Nat.3, Nat.5, Nat.6)
        four_edges_from_zero_share_color(color)
        }
        } else {
        if color(Nat.0, Nat.8) {
        color(Nat.0, Nat.1) = color(Nat.0, Nat.3)
        color(Nat.0, Nat.3) = color(Nat.0, Nat.5)
        color(Nat.0, Nat.5) = color(Nat.0, Nat.6)
        good_four_1356
        good_four(Nat.1, Nat.3, Nat.5, Nat.6)
        four_edges_from_zero_share_color_intro(color, Nat.1, Nat.3, Nat.5, Nat.6)
        four_edges_from_zero_share_color(color)
        } else {
        color(Nat.0, Nat.1) = color(Nat.0, Nat.3)
        color(Nat.0, Nat.3) = color(Nat.0, Nat.5)
        color(Nat.0, Nat.5) = color(Nat.0, Nat.6)
        good_four_1356
        good_four(Nat.1, Nat.3, Nat.5, Nat.6)
        four_edges_from_zero_share_color_intro(color, Nat.1, Nat.3, Nat.5, Nat.6)
        four_edges_from_zero_share_color(color)
        }
        }
        }
        }
        } else {
        if color(Nat.0, Nat.5) {
        if color(Nat.0, Nat.6) {
        if color(Nat.0, Nat.7) {
        if color(Nat.0, Nat.8) {
        color(Nat.0, Nat.2) = color(Nat.0, Nat.5)
        color(Nat.0, Nat.5) = color(Nat.0, Nat.6)
        color(Nat.0, Nat.6) = color(Nat.0, Nat.7)
        good_four_2567
        good_four(Nat.2, Nat.5, Nat.6, Nat.7)
        four_edges_from_zero_share_color_intro(color, Nat.2, Nat.5, Nat.6, Nat.7)
        four_edges_from_zero_share_color(color)
        } else {
        color(Nat.0, Nat.2) = color(Nat.0, Nat.5)
        color(Nat.0, Nat.5) = color(Nat.0, Nat.6)
        color(Nat.0, Nat.6) = color(Nat.0, Nat.7)
        good_four_2567
        good_four(Nat.2, Nat.5, Nat.6, Nat.7)
        four_edges_from_zero_share_color_intro(color, Nat.2, Nat.5, Nat.6, Nat.7)
        four_edges_from_zero_share_color(color)
        }
        } else {
        if color(Nat.0, Nat.8) {
        color(Nat.0, Nat.2) = color(Nat.0, Nat.5)
        color(Nat.0, Nat.5) = color(Nat.0, Nat.6)
        color(Nat.0, Nat.6) = color(Nat.0, Nat.8)
        good_four_2568
        good_four(Nat.2, Nat.5, Nat.6, Nat.8)
        four_edges_from_zero_share_color_intro(color, Nat.2, Nat.5, Nat.6, Nat.8)
        four_edges_from_zero_share_color(color)
        } else {
        color(Nat.0, Nat.1) = color(Nat.0, Nat.3)
        color(Nat.0, Nat.3) = color(Nat.0, Nat.4)
        color(Nat.0, Nat.4) = color(Nat.0, Nat.7)
        good_four_1347
        good_four(Nat.1, Nat.3, Nat.4, Nat.7)
        four_edges_from_zero_share_color_intro(color, Nat.1, Nat.3, Nat.4, Nat.7)
        four_edges_from_zero_share_color(color)
        }
        }
        } else {
        if color(Nat.0, Nat.7) {
        if color(Nat.0, Nat.8) {
        color(Nat.0, Nat.2) = color(Nat.0, Nat.5)
        color(Nat.0, Nat.5) = color(Nat.0, Nat.7)
        color(Nat.0, Nat.7) = color(Nat.0, Nat.8)
        good_four_2578
        good_four(Nat.2, Nat.5, Nat.7, Nat.8)
        four_edges_from_zero_share_color_intro(color, Nat.2, Nat.5, Nat.7, Nat.8)
        four_edges_from_zero_share_color(color)
        } else {
        color(Nat.0, Nat.1) = color(Nat.0, Nat.3)
        color(Nat.0, Nat.3) = color(Nat.0, Nat.4)
        color(Nat.0, Nat.4) = color(Nat.0, Nat.6)
        good_four_1346
        good_four(Nat.1, Nat.3, Nat.4, Nat.6)
        four_edges_from_zero_share_color_intro(color, Nat.1, Nat.3, Nat.4, Nat.6)
        four_edges_from_zero_share_color(color)
        }
        } else {
        if color(Nat.0, Nat.8) {
        color(Nat.0, Nat.1) = color(Nat.0, Nat.3)
        color(Nat.0, Nat.3) = color(Nat.0, Nat.4)
        color(Nat.0, Nat.4) = color(Nat.0, Nat.6)
        good_four_1346
        good_four(Nat.1, Nat.3, Nat.4, Nat.6)
        four_edges_from_zero_share_color_intro(color, Nat.1, Nat.3, Nat.4, Nat.6)
        four_edges_from_zero_share_color(color)
        } else {
        color(Nat.0, Nat.1) = color(Nat.0, Nat.3)
        color(Nat.0, Nat.3) = color(Nat.0, Nat.4)
        color(Nat.0, Nat.4) = color(Nat.0, Nat.6)
        good_four_1346
        good_four(Nat.1, Nat.3, Nat.4, Nat.6)
        four_edges_from_zero_share_color_intro(color, Nat.1, Nat.3, Nat.4, Nat.6)
        four_edges_from_zero_share_color(color)
        }
        }
        }
        } else {
        if color(Nat.0, Nat.6) {
        if color(Nat.0, Nat.7) {
        if color(Nat.0, Nat.8) {
        color(Nat.0, Nat.2) = color(Nat.0, Nat.6)
        color(Nat.0, Nat.6) = color(Nat.0, Nat.7)
        color(Nat.0, Nat.7) = color(Nat.0, Nat.8)
        good_four_2678
        good_four(Nat.2, Nat.6, Nat.7, Nat.8)
        four_edges_from_zero_share_color_intro(color, Nat.2, Nat.6, Nat.7, Nat.8)
        four_edges_from_zero_share_color(color)
        } else {
        color(Nat.0, Nat.1) = color(Nat.0, Nat.3)
        color(Nat.0, Nat.3) = color(Nat.0, Nat.4)
        color(Nat.0, Nat.4) = color(Nat.0, Nat.5)
        good_four_1345
        good_four(Nat.1, Nat.3, Nat.4, Nat.5)
        four_edges_from_zero_share_color_intro(color, Nat.1, Nat.3, Nat.4, Nat.5)
        four_edges_from_zero_share_color(color)
        }
        } else {
        if color(Nat.0, Nat.8) {
        color(Nat.0, Nat.1) = color(Nat.0, Nat.3)
        color(Nat.0, Nat.3) = color(Nat.0, Nat.4)
        color(Nat.0, Nat.4) = color(Nat.0, Nat.5)
        good_four_1345
        good_four(Nat.1, Nat.3, Nat.4, Nat.5)
        four_edges_from_zero_share_color_intro(color, Nat.1, Nat.3, Nat.4, Nat.5)
        four_edges_from_zero_share_color(color)
        } else {
        color(Nat.0, Nat.1) = color(Nat.0, Nat.3)
        color(Nat.0, Nat.3) = color(Nat.0, Nat.4)
        color(Nat.0, Nat.4) = color(Nat.0, Nat.5)
        good_four_1345
        good_four(Nat.1, Nat.3, Nat.4, Nat.5)
        four_edges_from_zero_share_color_intro(color, Nat.1, Nat.3, Nat.4, Nat.5)
        four_edges_from_zero_share_color(color)
        }
        }
        } else {
        if color(Nat.0, Nat.7) {
        if color(Nat.0, Nat.8) {
        color(Nat.0, Nat.1) = color(Nat.0, Nat.3)
        color(Nat.0, Nat.3) = color(Nat.0, Nat.4)
        color(Nat.0, Nat.4) = color(Nat.0, Nat.5)
        good_four_1345
        good_four(Nat.1, Nat.3, Nat.4, Nat.5)
        four_edges_from_zero_share_color_intro(color, Nat.1, Nat.3, Nat.4, Nat.5)
        four_edges_from_zero_share_color(color)
        } else {
        color(Nat.0, Nat.1) = color(Nat.0, Nat.3)
        color(Nat.0, Nat.3) = color(Nat.0, Nat.4)
        color(Nat.0, Nat.4) = color(Nat.0, Nat.5)
        good_four_1345
        good_four(Nat.1, Nat.3, Nat.4, Nat.5)
        four_edges_from_zero_share_color_intro(color, Nat.1, Nat.3, Nat.4, Nat.5)
        four_edges_from_zero_share_color(color)
        }
        } else {
        if color(Nat.0, Nat.8) {
        color(Nat.0, Nat.1) = color(Nat.0, Nat.3)
        color(Nat.0, Nat.3) = color(Nat.0, Nat.4)
        color(Nat.0, Nat.4) = color(Nat.0, Nat.5)
        good_four_1345
        good_four(Nat.1, Nat.3, Nat.4, Nat.5)
        four_edges_from_zero_share_color_intro(color, Nat.1, Nat.3, Nat.4, Nat.5)
        four_edges_from_zero_share_color(color)
        } else {
        color(Nat.0, Nat.1) = color(Nat.0, Nat.3)
        color(Nat.0, Nat.3) = color(Nat.0, Nat.4)
        color(Nat.0, Nat.4) = color(Nat.0, Nat.5)
        good_four_1345
        good_four(Nat.1, Nat.3, Nat.4, Nat.5)
        four_edges_from_zero_share_color_intro(color, Nat.1, Nat.3, Nat.4, Nat.5)
        four_edges_from_zero_share_color(color)
        }
        }
        }
        }
        }
        }
        } else {
        if color(Nat.0, Nat.3) {
        if color(Nat.0, Nat.4) {
        if color(Nat.0, Nat.5) {
        if color(Nat.0, Nat.6) {
        if color(Nat.0, Nat.7) {
        if color(Nat.0, Nat.8) {
        color(Nat.0, Nat.3) = color(Nat.0, Nat.4)
        color(Nat.0, Nat.4) = color(Nat.0, Nat.5)
        color(Nat.0, Nat.5) = color(Nat.0, Nat.6)
        good_four_3456
        good_four(Nat.3, Nat.4, Nat.5, Nat.6)
        four_edges_from_zero_share_color_intro(color, Nat.3, Nat.4, Nat.5, Nat.6)
        four_edges_from_zero_share_color(color)
        } else {
        color(Nat.0, Nat.3) = color(Nat.0, Nat.4)
        color(Nat.0, Nat.4) = color(Nat.0, Nat.5)
        color(Nat.0, Nat.5) = color(Nat.0, Nat.6)
        good_four_3456
        good_four(Nat.3, Nat.4, Nat.5, Nat.6)
        four_edges_from_zero_share_color_intro(color, Nat.3, Nat.4, Nat.5, Nat.6)
        four_edges_from_zero_share_color(color)
        }
        } else {
        if color(Nat.0, Nat.8) {
        color(Nat.0, Nat.3) = color(Nat.0, Nat.4)
        color(Nat.0, Nat.4) = color(Nat.0, Nat.5)
        color(Nat.0, Nat.5) = color(Nat.0, Nat.6)
        good_four_3456
        good_four(Nat.3, Nat.4, Nat.5, Nat.6)
        four_edges_from_zero_share_color_intro(color, Nat.3, Nat.4, Nat.5, Nat.6)
        four_edges_from_zero_share_color(color)
        } else {
        color(Nat.0, Nat.3) = color(Nat.0, Nat.4)
        color(Nat.0, Nat.4) = color(Nat.0, Nat.5)
        color(Nat.0, Nat.5) = color(Nat.0, Nat.6)
        good_four_3456
        good_four(Nat.3, Nat.4, Nat.5, Nat.6)
        four_edges_from_zero_share_color_intro(color, Nat.3, Nat.4, Nat.5, Nat.6)
        four_edges_from_zero_share_color(color)
        }
        }
        } else {
        if color(Nat.0, Nat.7) {
        if color(Nat.0, Nat.8) {
        color(Nat.0, Nat.3) = color(Nat.0, Nat.4)
        color(Nat.0, Nat.4) = color(Nat.0, Nat.5)
        color(Nat.0, Nat.5) = color(Nat.0, Nat.7)
        good_four_3457
        good_four(Nat.3, Nat.4, Nat.5, Nat.7)
        four_edges_from_zero_share_color_intro(color, Nat.3, Nat.4, Nat.5, Nat.7)
        four_edges_from_zero_share_color(color)
        } else {
        color(Nat.0, Nat.3) = color(Nat.0, Nat.4)
        color(Nat.0, Nat.4) = color(Nat.0, Nat.5)
        color(Nat.0, Nat.5) = color(Nat.0, Nat.7)
        good_four_3457
        good_four(Nat.3, Nat.4, Nat.5, Nat.7)
        four_edges_from_zero_share_color_intro(color, Nat.3, Nat.4, Nat.5, Nat.7)
        four_edges_from_zero_share_color(color)
        }
        } else {
        if color(Nat.0, Nat.8) {
        color(Nat.0, Nat.3) = color(Nat.0, Nat.4)
        color(Nat.0, Nat.4) = color(Nat.0, Nat.5)
        color(Nat.0, Nat.5) = color(Nat.0, Nat.8)
        good_four_3458
        good_four(Nat.3, Nat.4, Nat.5, Nat.8)
        four_edges_from_zero_share_color_intro(color, Nat.3, Nat.4, Nat.5, Nat.8)
        four_edges_from_zero_share_color(color)
        } else {
        color(Nat.0, Nat.1) = color(Nat.0, Nat.2)
        color(Nat.0, Nat.2) = color(Nat.0, Nat.6)
        color(Nat.0, Nat.6) = color(Nat.0, Nat.7)
        good_four_1267
        good_four(Nat.1, Nat.2, Nat.6, Nat.7)
        four_edges_from_zero_share_color_intro(color, Nat.1, Nat.2, Nat.6, Nat.7)
        four_edges_from_zero_share_color(color)
        }
        }
        }
        } else {
        if color(Nat.0, Nat.6) {
        if color(Nat.0, Nat.7) {
        if color(Nat.0, Nat.8) {
        color(Nat.0, Nat.3) = color(Nat.0, Nat.4)
        color(Nat.0, Nat.4) = color(Nat.0, Nat.6)
        color(Nat.0, Nat.6) = color(Nat.0, Nat.7)
        good_four_3467
        good_four(Nat.3, Nat.4, Nat.6, Nat.7)
        four_edges_from_zero_share_color_intro(color, Nat.3, Nat.4, Nat.6, Nat.7)
        four_edges_from_zero_share_color(color)
        } else {
        color(Nat.0, Nat.3) = color(Nat.0, Nat.4)
        color(Nat.0, Nat.4) = color(Nat.0, Nat.6)
        color(Nat.0, Nat.6) = color(Nat.0, Nat.7)
        good_four_3467
        good_four(Nat.3, Nat.4, Nat.6, Nat.7)
        four_edges_from_zero_share_color_intro(color, Nat.3, Nat.4, Nat.6, Nat.7)
        four_edges_from_zero_share_color(color)
        }
        } else {
        if color(Nat.0, Nat.8) {
        color(Nat.0, Nat.3) = color(Nat.0, Nat.4)
        color(Nat.0, Nat.4) = color(Nat.0, Nat.6)
        color(Nat.0, Nat.6) = color(Nat.0, Nat.8)
        good_four_3468
        good_four(Nat.3, Nat.4, Nat.6, Nat.8)
        four_edges_from_zero_share_color_intro(color, Nat.3, Nat.4, Nat.6, Nat.8)
        four_edges_from_zero_share_color(color)
        } else {
        color(Nat.0, Nat.1) = color(Nat.0, Nat.2)
        color(Nat.0, Nat.2) = color(Nat.0, Nat.5)
        color(Nat.0, Nat.5) = color(Nat.0, Nat.7)
        good_four_1257
        good_four(Nat.1, Nat.2, Nat.5, Nat.7)
        four_edges_from_zero_share_color_intro(color, Nat.1, Nat.2, Nat.5, Nat.7)
        four_edges_from_zero_share_color(color)
        }
        }
        } else {
        if color(Nat.0, Nat.7) {
        if color(Nat.0, Nat.8) {
        color(Nat.0, Nat.3) = color(Nat.0, Nat.4)
        color(Nat.0, Nat.4) = color(Nat.0, Nat.7)
        color(Nat.0, Nat.7) = color(Nat.0, Nat.8)
        good_four_3478
        good_four(Nat.3, Nat.4, Nat.7, Nat.8)
        four_edges_from_zero_share_color_intro(color, Nat.3, Nat.4, Nat.7, Nat.8)
        four_edges_from_zero_share_color(color)
        } else {
        color(Nat.0, Nat.1) = color(Nat.0, Nat.2)
        color(Nat.0, Nat.2) = color(Nat.0, Nat.5)
        color(Nat.0, Nat.5) = color(Nat.0, Nat.6)
        good_four_1256
        good_four(Nat.1, Nat.2, Nat.5, Nat.6)
        four_edges_from_zero_share_color_intro(color, Nat.1, Nat.2, Nat.5, Nat.6)
        four_edges_from_zero_share_color(color)
        }
        } else {
        if color(Nat.0, Nat.8) {
        color(Nat.0, Nat.1) = color(Nat.0, Nat.2)
        color(Nat.0, Nat.2) = color(Nat.0, Nat.5)
        color(Nat.0, Nat.5) = color(Nat.0, Nat.6)
        good_four_1256
        good_four(Nat.1, Nat.2, Nat.5, Nat.6)
        four_edges_from_zero_share_color_intro(color, Nat.1, Nat.2, Nat.5, Nat.6)
        four_edges_from_zero_share_color(color)
        } else {
        color(Nat.0, Nat.1) = color(Nat.0, Nat.2)
        color(Nat.0, Nat.2) = color(Nat.0, Nat.5)
        color(Nat.0, Nat.5) = color(Nat.0, Nat.6)
        good_four_1256
        good_four(Nat.1, Nat.2, Nat.5, Nat.6)
        four_edges_from_zero_share_color_intro(color, Nat.1, Nat.2, Nat.5, Nat.6)
        four_edges_from_zero_share_color(color)
        }
        }
        }
        }
        } else {
        if color(Nat.0, Nat.5) {
        if color(Nat.0, Nat.6) {
        if color(Nat.0, Nat.7) {
        if color(Nat.0, Nat.8) {
        color(Nat.0, Nat.3) = color(Nat.0, Nat.5)
        color(Nat.0, Nat.5) = color(Nat.0, Nat.6)
        color(Nat.0, Nat.6) = color(Nat.0, Nat.7)
        good_four_3567
        good_four(Nat.3, Nat.5, Nat.6, Nat.7)
        four_edges_from_zero_share_color_intro(color, Nat.3, Nat.5, Nat.6, Nat.7)
        four_edges_from_zero_share_color(color)
        } else {
        color(Nat.0, Nat.3) = color(Nat.0, Nat.5)
        color(Nat.0, Nat.5) = color(Nat.0, Nat.6)
        color(Nat.0, Nat.6) = color(Nat.0, Nat.7)
        good_four_3567
        good_four(Nat.3, Nat.5, Nat.6, Nat.7)
        four_edges_from_zero_share_color_intro(color, Nat.3, Nat.5, Nat.6, Nat.7)
        four_edges_from_zero_share_color(color)
        }
        } else {
        if color(Nat.0, Nat.8) {
        color(Nat.0, Nat.3) = color(Nat.0, Nat.5)
        color(Nat.0, Nat.5) = color(Nat.0, Nat.6)
        color(Nat.0, Nat.6) = color(Nat.0, Nat.8)
        good_four_3568
        good_four(Nat.3, Nat.5, Nat.6, Nat.8)
        four_edges_from_zero_share_color_intro(color, Nat.3, Nat.5, Nat.6, Nat.8)
        four_edges_from_zero_share_color(color)
        } else {
        color(Nat.0, Nat.1) = color(Nat.0, Nat.2)
        color(Nat.0, Nat.2) = color(Nat.0, Nat.4)
        color(Nat.0, Nat.4) = color(Nat.0, Nat.7)
        good_four_1247
        good_four(Nat.1, Nat.2, Nat.4, Nat.7)
        four_edges_from_zero_share_color_intro(color, Nat.1, Nat.2, Nat.4, Nat.7)
        four_edges_from_zero_share_color(color)
        }
        }
        } else {
        if color(Nat.0, Nat.7) {
        if color(Nat.0, Nat.8) {
        color(Nat.0, Nat.3) = color(Nat.0, Nat.5)
        color(Nat.0, Nat.5) = color(Nat.0, Nat.7)
        color(Nat.0, Nat.7) = color(Nat.0, Nat.8)
        good_four_3578
        good_four(Nat.3, Nat.5, Nat.7, Nat.8)
        four_edges_from_zero_share_color_intro(color, Nat.3, Nat.5, Nat.7, Nat.8)
        four_edges_from_zero_share_color(color)
        } else {
        color(Nat.0, Nat.1) = color(Nat.0, Nat.2)
        color(Nat.0, Nat.2) = color(Nat.0, Nat.4)
        color(Nat.0, Nat.4) = color(Nat.0, Nat.6)
        good_four_1246
        good_four(Nat.1, Nat.2, Nat.4, Nat.6)
        four_edges_from_zero_share_color_intro(color, Nat.1, Nat.2, Nat.4, Nat.6)
        four_edges_from_zero_share_color(color)
        }
        } else {
        if color(Nat.0, Nat.8) {
        color(Nat.0, Nat.1) = color(Nat.0, Nat.2)
        color(Nat.0, Nat.2) = color(Nat.0, Nat.4)
        color(Nat.0, Nat.4) = color(Nat.0, Nat.6)
        good_four_1246
        good_four(Nat.1, Nat.2, Nat.4, Nat.6)
        four_edges_from_zero_share_color_intro(color, Nat.1, Nat.2, Nat.4, Nat.6)
        four_edges_from_zero_share_color(color)
        } else {
        color(Nat.0, Nat.1) = color(Nat.0, Nat.2)
        color(Nat.0, Nat.2) = color(Nat.0, Nat.4)
        color(Nat.0, Nat.4) = color(Nat.0, Nat.6)
        good_four_1246
        good_four(Nat.1, Nat.2, Nat.4, Nat.6)
        four_edges_from_zero_share_color_intro(color, Nat.1, Nat.2, Nat.4, Nat.6)
        four_edges_from_zero_share_color(color)
        }
        }
        }
        } else {
        if color(Nat.0, Nat.6) {
        if color(Nat.0, Nat.7) {
        if color(Nat.0, Nat.8) {
        color(Nat.0, Nat.3) = color(Nat.0, Nat.6)
        color(Nat.0, Nat.6) = color(Nat.0, Nat.7)
        color(Nat.0, Nat.7) = color(Nat.0, Nat.8)
        good_four_3678
        good_four(Nat.3, Nat.6, Nat.7, Nat.8)
        four_edges_from_zero_share_color_intro(color, Nat.3, Nat.6, Nat.7, Nat.8)
        four_edges_from_zero_share_color(color)
        } else {
        color(Nat.0, Nat.1) = color(Nat.0, Nat.2)
        color(Nat.0, Nat.2) = color(Nat.0, Nat.4)
        color(Nat.0, Nat.4) = color(Nat.0, Nat.5)
        good_four_1245
        good_four(Nat.1, Nat.2, Nat.4, Nat.5)
        four_edges_from_zero_share_color_intro(color, Nat.1, Nat.2, Nat.4, Nat.5)
        four_edges_from_zero_share_color(color)
        }
        } else {
        if color(Nat.0, Nat.8) {
        color(Nat.0, Nat.1) = color(Nat.0, Nat.2)
        color(Nat.0, Nat.2) = color(Nat.0, Nat.4)
        color(Nat.0, Nat.4) = color(Nat.0, Nat.5)
        good_four_1245
        good_four(Nat.1, Nat.2, Nat.4, Nat.5)
        four_edges_from_zero_share_color_intro(color, Nat.1, Nat.2, Nat.4, Nat.5)
        four_edges_from_zero_share_color(color)
        } else {
        color(Nat.0, Nat.1) = color(Nat.0, Nat.2)
        color(Nat.0, Nat.2) = color(Nat.0, Nat.4)
        color(Nat.0, Nat.4) = color(Nat.0, Nat.5)
        good_four_1245
        good_four(Nat.1, Nat.2, Nat.4, Nat.5)
        four_edges_from_zero_share_color_intro(color, Nat.1, Nat.2, Nat.4, Nat.5)
        four_edges_from_zero_share_color(color)
        }
        }
        } else {
        if color(Nat.0, Nat.7) {
        if color(Nat.0, Nat.8) {
        color(Nat.0, Nat.1) = color(Nat.0, Nat.2)
        color(Nat.0, Nat.2) = color(Nat.0, Nat.4)
        color(Nat.0, Nat.4) = color(Nat.0, Nat.5)
        good_four_1245
        good_four(Nat.1, Nat.2, Nat.4, Nat.5)
        four_edges_from_zero_share_color_intro(color, Nat.1, Nat.2, Nat.4, Nat.5)
        four_edges_from_zero_share_color(color)
        } else {
        color(Nat.0, Nat.1) = color(Nat.0, Nat.2)
        color(Nat.0, Nat.2) = color(Nat.0, Nat.4)
        color(Nat.0, Nat.4) = color(Nat.0, Nat.5)
        good_four_1245
        good_four(Nat.1, Nat.2, Nat.4, Nat.5)
        four_edges_from_zero_share_color_intro(color, Nat.1, Nat.2, Nat.4, Nat.5)
        four_edges_from_zero_share_color(color)
        }
        } else {
        if color(Nat.0, Nat.8) {
        color(Nat.0, Nat.1) = color(Nat.0, Nat.2)
        color(Nat.0, Nat.2) = color(Nat.0, Nat.4)
        color(Nat.0, Nat.4) = color(Nat.0, Nat.5)
        good_four_1245
        good_four(Nat.1, Nat.2, Nat.4, Nat.5)
        four_edges_from_zero_share_color_intro(color, Nat.1, Nat.2, Nat.4, Nat.5)
        four_edges_from_zero_share_color(color)
        } else {
        color(Nat.0, Nat.1) = color(Nat.0, Nat.2)
        color(Nat.0, Nat.2) = color(Nat.0, Nat.4)
        color(Nat.0, Nat.4) = color(Nat.0, Nat.5)
        good_four_1245
        good_four(Nat.1, Nat.2, Nat.4, Nat.5)
        four_edges_from_zero_share_color_intro(color, Nat.1, Nat.2, Nat.4, Nat.5)
        four_edges_from_zero_share_color(color)
        }
        }
        }
        }
        }
        } else {
        if color(Nat.0, Nat.4) {
        if color(Nat.0, Nat.5) {
        if color(Nat.0, Nat.6) {
        if color(Nat.0, Nat.7) {
        if color(Nat.0, Nat.8) {
        color(Nat.0, Nat.4) = color(Nat.0, Nat.5)
        color(Nat.0, Nat.5) = color(Nat.0, Nat.6)
        color(Nat.0, Nat.6) = color(Nat.0, Nat.7)
        good_four_4567
        good_four(Nat.4, Nat.5, Nat.6, Nat.7)
        four_edges_from_zero_share_color_intro(color, Nat.4, Nat.5, Nat.6, Nat.7)
        four_edges_from_zero_share_color(color)
        } else {
        color(Nat.0, Nat.4) = color(Nat.0, Nat.5)
        color(Nat.0, Nat.5) = color(Nat.0, Nat.6)
        color(Nat.0, Nat.6) = color(Nat.0, Nat.7)
        good_four_4567
        good_four(Nat.4, Nat.5, Nat.6, Nat.7)
        four_edges_from_zero_share_color_intro(color, Nat.4, Nat.5, Nat.6, Nat.7)
        four_edges_from_zero_share_color(color)
        }
        } else {
        if color(Nat.0, Nat.8) {
        color(Nat.0, Nat.4) = color(Nat.0, Nat.5)
        color(Nat.0, Nat.5) = color(Nat.0, Nat.6)
        color(Nat.0, Nat.6) = color(Nat.0, Nat.8)
        good_four_4568
        good_four(Nat.4, Nat.5, Nat.6, Nat.8)
        four_edges_from_zero_share_color_intro(color, Nat.4, Nat.5, Nat.6, Nat.8)
        four_edges_from_zero_share_color(color)
        } else {
        color(Nat.0, Nat.1) = color(Nat.0, Nat.2)
        color(Nat.0, Nat.2) = color(Nat.0, Nat.3)
        color(Nat.0, Nat.3) = color(Nat.0, Nat.7)
        good_four_1237
        good_four(Nat.1, Nat.2, Nat.3, Nat.7)
        four_edges_from_zero_share_color_intro(color, Nat.1, Nat.2, Nat.3, Nat.7)
        four_edges_from_zero_share_color(color)
        }
        }
        } else {
        if color(Nat.0, Nat.7) {
        if color(Nat.0, Nat.8) {
        color(Nat.0, Nat.4) = color(Nat.0, Nat.5)
        color(Nat.0, Nat.5) = color(Nat.0, Nat.7)
        color(Nat.0, Nat.7) = color(Nat.0, Nat.8)
        good_four_4578
        good_four(Nat.4, Nat.5, Nat.7, Nat.8)
        four_edges_from_zero_share_color_intro(color, Nat.4, Nat.5, Nat.7, Nat.8)
        four_edges_from_zero_share_color(color)
        } else {
        color(Nat.0, Nat.1) = color(Nat.0, Nat.2)
        color(Nat.0, Nat.2) = color(Nat.0, Nat.3)
        color(Nat.0, Nat.3) = color(Nat.0, Nat.6)
        good_four_1236
        good_four(Nat.1, Nat.2, Nat.3, Nat.6)
        four_edges_from_zero_share_color_intro(color, Nat.1, Nat.2, Nat.3, Nat.6)
        four_edges_from_zero_share_color(color)
        }
        } else {
        if color(Nat.0, Nat.8) {
        color(Nat.0, Nat.1) = color(Nat.0, Nat.2)
        color(Nat.0, Nat.2) = color(Nat.0, Nat.3)
        color(Nat.0, Nat.3) = color(Nat.0, Nat.6)
        good_four_1236
        good_four(Nat.1, Nat.2, Nat.3, Nat.6)
        four_edges_from_zero_share_color_intro(color, Nat.1, Nat.2, Nat.3, Nat.6)
        four_edges_from_zero_share_color(color)
        } else {
        color(Nat.0, Nat.1) = color(Nat.0, Nat.2)
        color(Nat.0, Nat.2) = color(Nat.0, Nat.3)
        color(Nat.0, Nat.3) = color(Nat.0, Nat.6)
        good_four_1236
        good_four(Nat.1, Nat.2, Nat.3, Nat.6)
        four_edges_from_zero_share_color_intro(color, Nat.1, Nat.2, Nat.3, Nat.6)
        four_edges_from_zero_share_color(color)
        }
        }
        }
        } else {
        if color(Nat.0, Nat.6) {
        if color(Nat.0, Nat.7) {
        if color(Nat.0, Nat.8) {
        color(Nat.0, Nat.4) = color(Nat.0, Nat.6)
        color(Nat.0, Nat.6) = color(Nat.0, Nat.7)
        color(Nat.0, Nat.7) = color(Nat.0, Nat.8)
        good_four_4678
        good_four(Nat.4, Nat.6, Nat.7, Nat.8)
        four_edges_from_zero_share_color_intro(color, Nat.4, Nat.6, Nat.7, Nat.8)
        four_edges_from_zero_share_color(color)
        } else {
        color(Nat.0, Nat.1) = color(Nat.0, Nat.2)
        color(Nat.0, Nat.2) = color(Nat.0, Nat.3)
        color(Nat.0, Nat.3) = color(Nat.0, Nat.5)
        good_four_1235
        good_four(Nat.1, Nat.2, Nat.3, Nat.5)
        four_edges_from_zero_share_color_intro(color, Nat.1, Nat.2, Nat.3, Nat.5)
        four_edges_from_zero_share_color(color)
        }
        } else {
        if color(Nat.0, Nat.8) {
        color(Nat.0, Nat.1) = color(Nat.0, Nat.2)
        color(Nat.0, Nat.2) = color(Nat.0, Nat.3)
        color(Nat.0, Nat.3) = color(Nat.0, Nat.5)
        good_four_1235
        good_four(Nat.1, Nat.2, Nat.3, Nat.5)
        four_edges_from_zero_share_color_intro(color, Nat.1, Nat.2, Nat.3, Nat.5)
        four_edges_from_zero_share_color(color)
        } else {
        color(Nat.0, Nat.1) = color(Nat.0, Nat.2)
        color(Nat.0, Nat.2) = color(Nat.0, Nat.3)
        color(Nat.0, Nat.3) = color(Nat.0, Nat.5)
        good_four_1235
        good_four(Nat.1, Nat.2, Nat.3, Nat.5)
        four_edges_from_zero_share_color_intro(color, Nat.1, Nat.2, Nat.3, Nat.5)
        four_edges_from_zero_share_color(color)
        }
        }
        } else {
        if color(Nat.0, Nat.7) {
        if color(Nat.0, Nat.8) {
        color(Nat.0, Nat.1) = color(Nat.0, Nat.2)
        color(Nat.0, Nat.2) = color(Nat.0, Nat.3)
        color(Nat.0, Nat.3) = color(Nat.0, Nat.5)
        good_four_1235
        good_four(Nat.1, Nat.2, Nat.3, Nat.5)
        four_edges_from_zero_share_color_intro(color, Nat.1, Nat.2, Nat.3, Nat.5)
        four_edges_from_zero_share_color(color)
        } else {
        color(Nat.0, Nat.1) = color(Nat.0, Nat.2)
        color(Nat.0, Nat.2) = color(Nat.0, Nat.3)
        color(Nat.0, Nat.3) = color(Nat.0, Nat.5)
        good_four_1235
        good_four(Nat.1, Nat.2, Nat.3, Nat.5)
        four_edges_from_zero_share_color_intro(color, Nat.1, Nat.2, Nat.3, Nat.5)
        four_edges_from_zero_share_color(color)
        }
        } else {
        if color(Nat.0, Nat.8) {
        color(Nat.0, Nat.1) = color(Nat.0, Nat.2)
        color(Nat.0, Nat.2) = color(Nat.0, Nat.3)
        color(Nat.0, Nat.3) = color(Nat.0, Nat.5)
        good_four_1235
        good_four(Nat.1, Nat.2, Nat.3, Nat.5)
        four_edges_from_zero_share_color_intro(color, Nat.1, Nat.2, Nat.3, Nat.5)
        four_edges_from_zero_share_color(color)
        } else {
        color(Nat.0, Nat.1) = color(Nat.0, Nat.2)
        color(Nat.0, Nat.2) = color(Nat.0, Nat.3)
        color(Nat.0, Nat.3) = color(Nat.0, Nat.5)
        good_four_1235
        good_four(Nat.1, Nat.2, Nat.3, Nat.5)
        four_edges_from_zero_share_color_intro(color, Nat.1, Nat.2, Nat.3, Nat.5)
        four_edges_from_zero_share_color(color)
        }
        }
        }
        }
        } else {
        if color(Nat.0, Nat.5) {
        if color(Nat.0, Nat.6) {
        if color(Nat.0, Nat.7) {
        if color(Nat.0, Nat.8) {
        color(Nat.0, Nat.5) = color(Nat.0, Nat.6)
        color(Nat.0, Nat.6) = color(Nat.0, Nat.7)
        color(Nat.0, Nat.7) = color(Nat.0, Nat.8)
        good_four_5678
        good_four(Nat.5, Nat.6, Nat.7, Nat.8)
        four_edges_from_zero_share_color_intro(color, Nat.5, Nat.6, Nat.7, Nat.8)
        four_edges_from_zero_share_color(color)
        } else {
        color(Nat.0, Nat.1) = color(Nat.0, Nat.2)
        color(Nat.0, Nat.2) = color(Nat.0, Nat.3)
        color(Nat.0, Nat.3) = color(Nat.0, Nat.4)
        good_four_1234
        good_four(Nat.1, Nat.2, Nat.3, Nat.4)
        four_edges_from_zero_share_color_intro(color, Nat.1, Nat.2, Nat.3, Nat.4)
        four_edges_from_zero_share_color(color)
        }
        } else {
        if color(Nat.0, Nat.8) {
        color(Nat.0, Nat.1) = color(Nat.0, Nat.2)
        color(Nat.0, Nat.2) = color(Nat.0, Nat.3)
        color(Nat.0, Nat.3) = color(Nat.0, Nat.4)
        good_four_1234
        good_four(Nat.1, Nat.2, Nat.3, Nat.4)
        four_edges_from_zero_share_color_intro(color, Nat.1, Nat.2, Nat.3, Nat.4)
        four_edges_from_zero_share_color(color)
        } else {
        color(Nat.0, Nat.1) = color(Nat.0, Nat.2)
        color(Nat.0, Nat.2) = color(Nat.0, Nat.3)
        color(Nat.0, Nat.3) = color(Nat.0, Nat.4)
        good_four_1234
        good_four(Nat.1, Nat.2, Nat.3, Nat.4)
        four_edges_from_zero_share_color_intro(color, Nat.1, Nat.2, Nat.3, Nat.4)
        four_edges_from_zero_share_color(color)
        }
        }
        } else {
        if color(Nat.0, Nat.7) {
        if color(Nat.0, Nat.8) {
        color(Nat.0, Nat.1) = color(Nat.0, Nat.2)
        color(Nat.0, Nat.2) = color(Nat.0, Nat.3)
        color(Nat.0, Nat.3) = color(Nat.0, Nat.4)
        good_four_1234
        good_four(Nat.1, Nat.2, Nat.3, Nat.4)
        four_edges_from_zero_share_color_intro(color, Nat.1, Nat.2, Nat.3, Nat.4)
        four_edges_from_zero_share_color(color)
        } else {
        color(Nat.0, Nat.1) = color(Nat.0, Nat.2)
        color(Nat.0, Nat.2) = color(Nat.0, Nat.3)
        color(Nat.0, Nat.3) = color(Nat.0, Nat.4)
        good_four_1234
        good_four(Nat.1, Nat.2, Nat.3, Nat.4)
        four_edges_from_zero_share_color_intro(color, Nat.1, Nat.2, Nat.3, Nat.4)
        four_edges_from_zero_share_color(color)
        }
        } else {
        if color(Nat.0, Nat.8) {
        color(Nat.0, Nat.1) = color(Nat.0, Nat.2)
        color(Nat.0, Nat.2) = color(Nat.0, Nat.3)
        color(Nat.0, Nat.3) = color(Nat.0, Nat.4)
        good_four_1234
        good_four(Nat.1, Nat.2, Nat.3, Nat.4)
        four_edges_from_zero_share_color_intro(color, Nat.1, Nat.2, Nat.3, Nat.4)
        four_edges_from_zero_share_color(color)
        } else {
        color(Nat.0, Nat.1) = color(Nat.0, Nat.2)
        color(Nat.0, Nat.2) = color(Nat.0, Nat.3)
        color(Nat.0, Nat.3) = color(Nat.0, Nat.4)
        good_four_1234
        good_four(Nat.1, Nat.2, Nat.3, Nat.4)
        four_edges_from_zero_share_color_intro(color, Nat.1, Nat.2, Nat.3, Nat.4)
        four_edges_from_zero_share_color(color)
        }
        }
        }
        } else {
        if color(Nat.0, Nat.6) {
        if color(Nat.0, Nat.7) {
        if color(Nat.0, Nat.8) {
        color(Nat.0, Nat.1) = color(Nat.0, Nat.2)
        color(Nat.0, Nat.2) = color(Nat.0, Nat.3)
        color(Nat.0, Nat.3) = color(Nat.0, Nat.4)
        good_four_1234
        good_four(Nat.1, Nat.2, Nat.3, Nat.4)
        four_edges_from_zero_share_color_intro(color, Nat.1, Nat.2, Nat.3, Nat.4)
        four_edges_from_zero_share_color(color)
        } else {
        color(Nat.0, Nat.1) = color(Nat.0, Nat.2)
        color(Nat.0, Nat.2) = color(Nat.0, Nat.3)
        color(Nat.0, Nat.3) = color(Nat.0, Nat.4)
        good_four_1234
        good_four(Nat.1, Nat.2, Nat.3, Nat.4)
        four_edges_from_zero_share_color_intro(color, Nat.1, Nat.2, Nat.3, Nat.4)
        four_edges_from_zero_share_color(color)
        }
        } else {
        if color(Nat.0, Nat.8) {
        color(Nat.0, Nat.1) = color(Nat.0, Nat.2)
        color(Nat.0, Nat.2) = color(Nat.0, Nat.3)
        color(Nat.0, Nat.3) = color(Nat.0, Nat.4)
        good_four_1234
        good_four(Nat.1, Nat.2, Nat.3, Nat.4)
        four_edges_from_zero_share_color_intro(color, Nat.1, Nat.2, Nat.3, Nat.4)
        four_edges_from_zero_share_color(color)
        } else {
        color(Nat.0, Nat.1) = color(Nat.0, Nat.2)
        color(Nat.0, Nat.2) = color(Nat.0, Nat.3)
        color(Nat.0, Nat.3) = color(Nat.0, Nat.4)
        good_four_1234
        good_four(Nat.1, Nat.2, Nat.3, Nat.4)
        four_edges_from_zero_share_color_intro(color, Nat.1, Nat.2, Nat.3, Nat.4)
        four_edges_from_zero_share_color(color)
        }
        }
        } else {
        if color(Nat.0, Nat.7) {
        if color(Nat.0, Nat.8) {
        color(Nat.0, Nat.1) = color(Nat.0, Nat.2)
        color(Nat.0, Nat.2) = color(Nat.0, Nat.3)
        color(Nat.0, Nat.3) = color(Nat.0, Nat.4)
        good_four_1234
        good_four(Nat.1, Nat.2, Nat.3, Nat.4)
        four_edges_from_zero_share_color_intro(color, Nat.1, Nat.2, Nat.3, Nat.4)
        four_edges_from_zero_share_color(color)
        } else {
        color(Nat.0, Nat.1) = color(Nat.0, Nat.2)
        color(Nat.0, Nat.2) = color(Nat.0, Nat.3)
        color(Nat.0, Nat.3) = color(Nat.0, Nat.4)
        good_four_1234
        good_four(Nat.1, Nat.2, Nat.3, Nat.4)
        four_edges_from_zero_share_color_intro(color, Nat.1, Nat.2, Nat.3, Nat.4)
        four_edges_from_zero_share_color(color)
        }
        } else {
        if color(Nat.0, Nat.8) {
        color(Nat.0, Nat.1) = color(Nat.0, Nat.2)
        color(Nat.0, Nat.2) = color(Nat.0, Nat.3)
        color(Nat.0, Nat.3) = color(Nat.0, Nat.4)
        good_four_1234
        good_four(Nat.1, Nat.2, Nat.3, Nat.4)
        four_edges_from_zero_share_color_intro(color, Nat.1, Nat.2, Nat.3, Nat.4)
        four_edges_from_zero_share_color(color)
        } else {
        color(Nat.0, Nat.1) = color(Nat.0, Nat.2)
        color(Nat.0, Nat.2) = color(Nat.0, Nat.3)
        color(Nat.0, Nat.3) = color(Nat.0, Nat.4)
        good_four_1234
        good_four(Nat.1, Nat.2, Nat.3, Nat.4)
        four_edges_from_zero_share_color_intro(color, Nat.1, Nat.2, Nat.3, Nat.4)
        four_edges_from_zero_share_color(color)
        }
        }
        }
        }
        }
        }
        }
        }
}

/// The pigeonhole conclusion can be unpacked.
theorem four_edges_from_zero_share_color_witness(color: (Nat, Nat) -> Bool) {
    four_edges_from_zero_share_color(color) implies exists(w: Nat, x: Nat, y: Nat, z: Nat) {
        good_four(w, x, y, z)
        and color(Nat.0, w) = color(Nat.0, x) and color(Nat.0, x) = color(Nat.0, y)
        and color(Nat.0, y) = color(Nat.0, z)
    }
} by {
    if four_edges_from_zero_share_color(color) {
        four_edges_from_zero_share_color(color) = exists(a: Nat, b: Nat, c: Nat, d: Nat) {
            good_four(a, b, c, d)
            and color(Nat.0, a) = color(Nat.0, b) and color(Nat.0, b) = color(Nat.0, c)
            and color(Nat.0, c) = color(Nat.0, d)
        }
        exists(a: Nat, b: Nat, c: Nat, d: Nat) {
            good_four(a, b, c, d)
            and color(Nat.0, a) = color(Nat.0, b) and color(Nat.0, b) = color(Nat.0, c)
            and color(Nat.0, c) = color(Nat.0, d)
        }
    }
}
