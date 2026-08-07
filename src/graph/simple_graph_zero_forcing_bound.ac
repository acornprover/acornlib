from nat import Nat, lte_and_lt, lt_and_lte, lte_antisymm, lt_or_lte, lte_trans,
    lt_imp_lte_suc, only_zero_lte_zero
from finite_set import FiniteSet, finite_set_subset_refl
from data.finite.finite_set_card import fs_card, fs_card_cardinality_is
from data.finite.finite_set_card_compare import fs_card_mono
from data.finite.finite_set_subset_cardinality import finite_set_subset_eq_of_same_cardinality
from graph.simple_graph import SimpleGraph
from graph.simple_graph_zero_forcing_rule import derived_set, derived_set_subset, derived_set_grows
from graph.simple_graph_zero_forcing_closure import forcing_iterate, forcing_iterate_zero,
    forcing_iterate_suc, forcing_iterate_subset, forcing_iterate_step_grows

numerals Nat

/// A nested pair of finite sets with the same size is equal.
///
/// The numeric-cardinality form of `finite_set_subset_eq_of_same_cardinality`, which is stated
/// through the `cardinality_is` predicate.
theorem fs_subset_eq_of_same_card[T](s: FiniteSet[T], t: FiniteSet[T]) {
    s.subset_eq(t) and fs_card(s) = fs_card(t) implies s = t
} by {
    if s.subset_eq(t) and fs_card(s) = fs_card(t) {
        fs_card_cardinality_is(s)
        s.cardinality_is(fs_card(s))
        fs_card_cardinality_is(t)
        t.cardinality_is(fs_card(t))
        t.cardinality_is(fs_card(s))
        finite_set_subset_eq_of_same_cardinality(s, t, fs_card(s))
        s = t
    }
}

/// A proper enlargement of a finite set is strictly larger.
theorem fs_card_lt_of_proper_subset[T](s: FiniteSet[T], t: FiniteSet[T]) {
    s.subset_eq(t) and s != t implies fs_card(s) < fs_card(t)
} by {
    if s.subset_eq(t) and s != t {
        fs_card_mono(s, t)
        fs_card(s) <= fs_card(t)
        if fs_card(s) = fs_card(t) {
            fs_subset_eq_of_same_card(s, t)
            s = t
            false
        }
        fs_card(s) != fs_card(t)
        fs_card(s) < fs_card(t)
    }
}

/// A round of forcing that changes anything strictly increases the count of blue vertices.
///
/// The blue set only grows, so a round that is not a fixed point is a proper enlargement.
theorem derived_set_card_grows[V](g: SimpleGraph[V], s: FiniteSet[V], b: FiniteSet[V]) {
    b.subset_eq(s) and derived_set(g, s, b) != b implies fs_card(b) < fs_card(derived_set(g, s, b))
} by {
    if b.subset_eq(s) and derived_set(g, s, b) != b {
        derived_set_grows(g, s, b)
        b.subset_eq(derived_set(g, s, b))
        b != derived_set(g, s, b)
        fs_card_lt_of_proper_subset(b, derived_set(g, s, b))
        fs_card(b) < fs_card(derived_set(g, s, b))
    }
}

/// True if no round below `n` leaves the colouring unchanged.
///
/// Named so the counting bound below is a statement about a single number rather than a
/// hypothesis repeated inside an induction.
define strictly_growing_upto[V](
    g: SimpleGraph[V], s: FiniteSet[V], b: FiniteSet[V], n: Nat
) -> Bool {
    forall(k: Nat) {
        k < n implies forcing_iterate(g, s, b, k.suc) != forcing_iterate(g, s, b, k)
    }
}

/// Each round below the bound changes the colouring.
theorem strictly_growing_upto_apply[V](
    g: SimpleGraph[V], s: FiniteSet[V], b: FiniteSet[V], n: Nat, k: Nat
) {
    strictly_growing_upto(g, s, b, n) and k < n
        implies forcing_iterate(g, s, b, k.suc) != forcing_iterate(g, s, b, k)
} by {
    if strictly_growing_upto(g, s, b, n) and k < n {
        strictly_growing_upto(g, s, b, n) = forall(j: Nat) {
            j < n implies forcing_iterate(g, s, b, j.suc) != forcing_iterate(g, s, b, j)
        }
        forall(j: Nat) {
            j < n implies forcing_iterate(g, s, b, j.suc) != forcing_iterate(g, s, b, j)
        }
        (k < n implies forcing_iterate(g, s, b, k.suc) != forcing_iterate(g, s, b, k))
        forcing_iterate(g, s, b, k.suc) != forcing_iterate(g, s, b, k)
    }
}

/// A shorter run of changing rounds is still a run of changing rounds.
theorem strictly_growing_upto_weaken[V](
    g: SimpleGraph[V], s: FiniteSet[V], b: FiniteSet[V], n: Nat, m: Nat
) {
    strictly_growing_upto(g, s, b, n) and m <= n implies strictly_growing_upto(g, s, b, m)
} by {
    if strictly_growing_upto(g, s, b, n) and m <= n {
        forall(k: Nat) {
            if k < m {
                lt_and_lte(k, m, n)
                k < n
                strictly_growing_upto_apply(g, s, b, n, k)
                forcing_iterate(g, s, b, k.suc) != forcing_iterate(g, s, b, k)
            }
            (k < m implies forcing_iterate(g, s, b, k.suc) != forcing_iterate(g, s, b, k))
        }
        strictly_growing_upto(g, s, b, m) = forall(k: Nat) {
            k < m implies forcing_iterate(g, s, b, k.suc) != forcing_iterate(g, s, b, k)
        }
        strictly_growing_upto(g, s, b, m)
    }
}

/// Every changing round adds at least one vertex, so `n` of them add at least `n`.
theorem forcing_iterate_card_bound[V](
    g: SimpleGraph[V], s: FiniteSet[V], b: FiniteSet[V], n: Nat
) {
    b.subset_eq(s) and strictly_growing_upto(g, s, b, n)
        implies fs_card(b) + n <= fs_card(forcing_iterate(g, s, b, n))
} by {
    if b.subset_eq(s) {
        define p(x: Nat) -> Bool {
            strictly_growing_upto(g, s, b, x)
                implies fs_card(b) + x <= fs_card(forcing_iterate(g, s, b, x))
        }
        forcing_iterate_zero(g, s, b)
        forcing_iterate(g, s, b, Nat.0) = b
        fs_card(b) + Nat.0 = fs_card(b)
        fs_card(b) + Nat.0 <= fs_card(forcing_iterate(g, s, b, Nat.0))
        p(Nat.0)
        forall(k: Nat) {
            if p(k) {
                if strictly_growing_upto(g, s, b, k.suc) {
                    k <= k.suc
                    strictly_growing_upto_weaken(g, s, b, k.suc, k)
                    strictly_growing_upto(g, s, b, k)
                    fs_card(b) + k <= fs_card(forcing_iterate(g, s, b, k))
                    k < k.suc
                    strictly_growing_upto_apply(g, s, b, k.suc, k)
                    forcing_iterate(g, s, b, k.suc) != forcing_iterate(g, s, b, k)
                    forcing_iterate_subset(g, s, b, k)
                    forcing_iterate(g, s, b, k).subset_eq(s)
                    forcing_iterate_suc(g, s, b, k)
                    (forcing_iterate(g, s, b, k.suc)
                        = derived_set(g, s, forcing_iterate(g, s, b, k)))
                    (derived_set(g, s, forcing_iterate(g, s, b, k))
                        != forcing_iterate(g, s, b, k))
                    derived_set_card_grows(g, s, forcing_iterate(g, s, b, k))
                    (fs_card(forcing_iterate(g, s, b, k))
                        < fs_card(derived_set(g, s, forcing_iterate(g, s, b, k))))
                    (fs_card(forcing_iterate(g, s, b, k))
                        < fs_card(forcing_iterate(g, s, b, k.suc)))
                    lt_imp_lte_suc(fs_card(forcing_iterate(g, s, b, k)),
                        fs_card(forcing_iterate(g, s, b, k.suc)))
                    (fs_card(forcing_iterate(g, s, b, k)).suc
                        <= fs_card(forcing_iterate(g, s, b, k.suc)))
                    (fs_card(b) + k).suc <= fs_card(forcing_iterate(g, s, b, k)).suc
                    (fs_card(b) + k).suc = fs_card(b) + k.suc
                    lte_trans(fs_card(b) + k.suc, fs_card(forcing_iterate(g, s, b, k)).suc,
                        fs_card(forcing_iterate(g, s, b, k.suc)))
                    fs_card(b) + k.suc <= fs_card(forcing_iterate(g, s, b, k.suc))
                }
                p(k.suc)
            }
            (p(k) implies p(k.suc))
        }
        p(Nat.0) and forall(k: Nat) {
            p(k) implies p(k.suc)
        }
        Nat.induction(p)
        p(n)
        (strictly_growing_upto(g, s, b, n)
            implies fs_card(b) + n <= fs_card(forcing_iterate(g, s, b, n)))
    }
}

/// The iteration cannot keep changing past the number of vertices.
///
/// Each changing round adds a vertex and the blue set never leaves the ambient set, so more
/// changing rounds than there are vertices is impossible. This is what makes the existential
/// in `is_zero_forcing_set` decidable at a fixed stage rather than unbounded.
theorem forcing_iterate_becomes_stationary[V](
    g: SimpleGraph[V], s: FiniteSet[V], b: FiniteSet[V]
) {
    b.subset_eq(s) implies not strictly_growing_upto(g, s, b, fs_card(s).suc)
} by {
    if b.subset_eq(s) {
        if strictly_growing_upto(g, s, b, fs_card(s).suc) {
            forcing_iterate_card_bound(g, s, b, fs_card(s).suc)
            (fs_card(b) + fs_card(s).suc
                <= fs_card(forcing_iterate(g, s, b, fs_card(s).suc)))
            forcing_iterate_subset(g, s, b, fs_card(s).suc)
            forcing_iterate(g, s, b, fs_card(s).suc).subset_eq(s)
            fs_card_mono(forcing_iterate(g, s, b, fs_card(s).suc), s)
            fs_card(forcing_iterate(g, s, b, fs_card(s).suc)) <= fs_card(s)
            lte_trans(fs_card(b) + fs_card(s).suc,
                fs_card(forcing_iterate(g, s, b, fs_card(s).suc)), fs_card(s))
            fs_card(b) + fs_card(s).suc <= fs_card(s)
            fs_card(s).suc <= fs_card(b) + fs_card(s).suc
            lte_trans(fs_card(s).suc, fs_card(b) + fs_card(s).suc, fs_card(s))
            fs_card(s).suc <= fs_card(s)
            fs_card(s) < fs_card(s).suc
            lte_and_lt(fs_card(s).suc, fs_card(s), fs_card(s).suc)
            fs_card(s).suc < fs_card(s).suc
            false
        }
        not strictly_growing_upto(g, s, b, fs_card(s).suc)
    }
}

/// Some round at or below the vertex count leaves the colouring unchanged.
///
/// The usable form: the search for a fixed point never has to look past `fs_card(s)` steps.
theorem forcing_iterate_fixed_point_exists[V](
    g: SimpleGraph[V], s: FiniteSet[V], b: FiniteSet[V]
) {
    b.subset_eq(s) implies exists(k: Nat) {
        k <= fs_card(s) and forcing_iterate(g, s, b, k.suc) = forcing_iterate(g, s, b, k)
    }
} by {
    if b.subset_eq(s) {
        forcing_iterate_becomes_stationary(g, s, b)
        not strictly_growing_upto(g, s, b, fs_card(s).suc)
        strictly_growing_upto(g, s, b, fs_card(s).suc) = forall(k: Nat) {
            k < fs_card(s).suc
                implies forcing_iterate(g, s, b, k.suc) != forcing_iterate(g, s, b, k)
        }
        not forall(k: Nat) {
            k < fs_card(s).suc
                implies forcing_iterate(g, s, b, k.suc) != forcing_iterate(g, s, b, k)
        }
        exists(k: Nat) {
            k < fs_card(s).suc
                and forcing_iterate(g, s, b, k.suc) = forcing_iterate(g, s, b, k)
        }
        let (k: Nat) satisfy {
            k < fs_card(s).suc
                and forcing_iterate(g, s, b, k.suc) = forcing_iterate(g, s, b, k)
        }
        lt_imp_lte_suc(k, fs_card(s).suc)
        k.suc <= fs_card(s).suc
        k <= fs_card(s)
        exists(j: Nat) {
            j <= fs_card(s) and forcing_iterate(g, s, b, j.suc) = forcing_iterate(g, s, b, j)
        }
    }
}
