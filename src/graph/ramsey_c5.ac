from nat import Nat, lt_and_lte
from finite_set import FiniteSet
from data.nat.nat_range_set import range_set

numerals Nat

/// The five vertices `{0, ..., 4}` on which the lower bound lives.
let five_vertices: FiniteSet[Nat] = range_set(Nat.5)

/// Red on the five cycle edges of C5: consecutive successors, plus the wrap-around 4 -- 0.
///
/// Outside `{0, ..., 4}` the relation is not the intended one, which costs nothing:
/// the vertex set is always supplied.
define c5_color(x: Nat, y: Nat) -> Bool {
    x.suc = y or y.suc = x or (x = Nat.4 and y = Nat.0) or (y = Nat.4 and x = Nat.0)
}

theorem five_labels_lt {
    Nat.0 < Nat.1 and Nat.1 < Nat.2 and Nat.2 < Nat.3 and Nat.3 < Nat.4 and Nat.4 < Nat.5
    and Nat.0 < Nat.5 and Nat.1 < Nat.5 and Nat.2 < Nat.5 and Nat.3 < Nat.5
    and Nat.2 < Nat.4
} by {
    Nat.0.suc = Nat.1
    Nat.0 < Nat.0.suc
    Nat.0 < Nat.1
    Nat.1.suc = Nat.2
    Nat.1 < Nat.1.suc
    Nat.1 < Nat.2
    Nat.2.suc = Nat.3
    Nat.2 < Nat.2.suc
    Nat.2 < Nat.3
    Nat.3.suc = Nat.4
    Nat.3 < Nat.3.suc
    Nat.3 < Nat.4
    Nat.4.suc = Nat.5
    Nat.4 < Nat.4.suc
    Nat.4 < Nat.5
    Nat.4 <= Nat.5
    lt_and_lte(Nat.3, Nat.4, Nat.5)
    Nat.3 < Nat.5
    Nat.3 <= Nat.5
    lt_and_lte(Nat.2, Nat.3, Nat.5)
    Nat.2 < Nat.5
    Nat.2 <= Nat.5
    lt_and_lte(Nat.1, Nat.2, Nat.5)
    Nat.1 < Nat.5
    Nat.1 <= Nat.5
    lt_and_lte(Nat.0, Nat.1, Nat.5)
    Nat.0 < Nat.5
    Nat.3 <= Nat.4
    lt_and_lte(Nat.2, Nat.3, Nat.4)
    Nat.2 < Nat.4
}

theorem five_labels_distinct {
    Nat.0 != Nat.1 and Nat.0 != Nat.2 and Nat.0 != Nat.3 and Nat.0 != Nat.4
    and Nat.1 != Nat.2 and Nat.1 != Nat.3 and Nat.1 != Nat.4
    and Nat.2 != Nat.3 and Nat.2 != Nat.4
    and Nat.3 != Nat.4
} by {
    five_labels_lt
    (Nat.0 < Nat.1 and Nat.1 < Nat.2 and Nat.2 < Nat.3 and Nat.3 < Nat.4 and Nat.4 < Nat.5
        and Nat.0 < Nat.5 and Nat.1 < Nat.5 and Nat.2 < Nat.5 and Nat.3 < Nat.5
        and Nat.2 < Nat.4)
    Nat.0 < Nat.1
    Nat.0 != Nat.1
    Nat.0 < Nat.2
    Nat.0 != Nat.2
    Nat.0 < Nat.3
    Nat.0 != Nat.3
    Nat.0 < Nat.4
    Nat.0 != Nat.4
    Nat.1 < Nat.2
    Nat.1 != Nat.2
    Nat.1 < Nat.3
    Nat.1 != Nat.3
    Nat.1 < Nat.4
    Nat.1 != Nat.4
    Nat.2 < Nat.3
    Nat.2 != Nat.3
    Nat.2 < Nat.4
    Nat.2 != Nat.4
    Nat.3 < Nat.4
    Nat.3 != Nat.4
}

/// The cycle edge 0 -- 1 is red.
theorem c5_red_0_1 {
    c5_color(Nat.0, Nat.1)
} by {
    Nat.0.suc = Nat.1
    c5_color(Nat.0, Nat.1) = (Nat.0.suc = Nat.1 or Nat.1.suc = Nat.0 or (Nat.0 = Nat.4 and Nat.1 = Nat.0) or (Nat.1 = Nat.4 and Nat.0 = Nat.0))
    c5_color(Nat.0, Nat.1)
}

/// The cycle edge 1 -- 2 is red.
theorem c5_red_1_2 {
    c5_color(Nat.1, Nat.2)
} by {
    Nat.1.suc = Nat.2
    c5_color(Nat.1, Nat.2) = (Nat.1.suc = Nat.2 or Nat.2.suc = Nat.1 or (Nat.1 = Nat.4 and Nat.2 = Nat.0) or (Nat.2 = Nat.4 and Nat.1 = Nat.0))
    c5_color(Nat.1, Nat.2)
}

/// The cycle edge 2 -- 3 is red.
theorem c5_red_2_3 {
    c5_color(Nat.2, Nat.3)
} by {
    Nat.2.suc = Nat.3
    c5_color(Nat.2, Nat.3) = (Nat.2.suc = Nat.3 or Nat.3.suc = Nat.2 or (Nat.2 = Nat.4 and Nat.3 = Nat.0) or (Nat.3 = Nat.4 and Nat.2 = Nat.0))
    c5_color(Nat.2, Nat.3)
}

/// The cycle edge 3 -- 4 is red.
theorem c5_red_3_4 {
    c5_color(Nat.3, Nat.4)
} by {
    Nat.3.suc = Nat.4
    c5_color(Nat.3, Nat.4) = (Nat.3.suc = Nat.4 or Nat.4.suc = Nat.3 or (Nat.3 = Nat.4 and Nat.4 = Nat.0) or (Nat.4 = Nat.4 and Nat.3 = Nat.0))
    c5_color(Nat.3, Nat.4)
}

/// The cycle edge 4 -- 0 is red.
theorem c5_red_4_0 {
    c5_color(Nat.4, Nat.0)
} by {
    (Nat.4 = Nat.4 and Nat.0 = Nat.0)
    c5_color(Nat.4, Nat.0) = (Nat.4.suc = Nat.0 or Nat.0.suc = Nat.4 or (Nat.4 = Nat.4 and Nat.0 = Nat.0) or (Nat.0 = Nat.4 and Nat.4 = Nat.0))
    c5_color(Nat.4, Nat.0)
}

/// The cycle edge 1 -- 0 is red.
theorem c5_red_1_0 {
    c5_color(Nat.1, Nat.0)
} by {
    Nat.0.suc = Nat.1
    c5_color(Nat.1, Nat.0) = (Nat.1.suc = Nat.0 or Nat.0.suc = Nat.1 or (Nat.1 = Nat.4 and Nat.0 = Nat.0) or (Nat.0 = Nat.4 and Nat.1 = Nat.0))
    c5_color(Nat.1, Nat.0)
}

/// The cycle edge 2 -- 1 is red.
theorem c5_red_2_1 {
    c5_color(Nat.2, Nat.1)
} by {
    Nat.1.suc = Nat.2
    c5_color(Nat.2, Nat.1) = (Nat.2.suc = Nat.1 or Nat.1.suc = Nat.2 or (Nat.2 = Nat.4 and Nat.1 = Nat.0) or (Nat.1 = Nat.4 and Nat.2 = Nat.0))
    c5_color(Nat.2, Nat.1)
}

/// The cycle edge 3 -- 2 is red.
theorem c5_red_3_2 {
    c5_color(Nat.3, Nat.2)
} by {
    Nat.2.suc = Nat.3
    c5_color(Nat.3, Nat.2) = (Nat.3.suc = Nat.2 or Nat.2.suc = Nat.3 or (Nat.3 = Nat.4 and Nat.2 = Nat.0) or (Nat.2 = Nat.4 and Nat.3 = Nat.0))
    c5_color(Nat.3, Nat.2)
}

/// The cycle edge 4 -- 3 is red.
theorem c5_red_4_3 {
    c5_color(Nat.4, Nat.3)
} by {
    Nat.3.suc = Nat.4
    c5_color(Nat.4, Nat.3) = (Nat.4.suc = Nat.3 or Nat.3.suc = Nat.4 or (Nat.4 = Nat.4 and Nat.3 = Nat.0) or (Nat.3 = Nat.4 and Nat.4 = Nat.0))
    c5_color(Nat.4, Nat.3)
}

/// The cycle edge 0 -- 4 is red.
theorem c5_red_0_4 {
    c5_color(Nat.0, Nat.4)
} by {
    (Nat.4 = Nat.4 and Nat.0 = Nat.0)
    c5_color(Nat.0, Nat.4) = (Nat.0.suc = Nat.4 or Nat.4.suc = Nat.0 or (Nat.0 = Nat.4 and Nat.4 = Nat.0) or (Nat.4 = Nat.4 and Nat.0 = Nat.0))
    c5_color(Nat.0, Nat.4)
}

/// The diagonal 0 -- 2 is blue.
theorem c5_diag_0_2 {
    not c5_color(Nat.0, Nat.2)
} by {
    if c5_color(Nat.0, Nat.2) {
        c5_color(Nat.0, Nat.2) = (Nat.0.suc = Nat.2 or Nat.2.suc = Nat.0 or (Nat.0 = Nat.4 and Nat.2 = Nat.0) or (Nat.2 = Nat.4 and Nat.0 = Nat.0))
        Nat.0.suc = Nat.1
        Nat.2.suc = Nat.3
        if Nat.0.suc = Nat.2 {
            Nat.0.suc = Nat.1
            Nat.1 = Nat.2
            Nat.1 != Nat.2
            false
        } else {
            if Nat.2.suc = Nat.0 {
                Nat.2.suc = Nat.3
                Nat.3 = Nat.0
                Nat.3 != Nat.0
                false
            } else {
                if Nat.0 = Nat.4 and Nat.2 = Nat.0 {
                    Nat.0 = Nat.4
                    Nat.2 = Nat.0
                    Nat.0 != Nat.4
                    false
                } else {
                    if Nat.2 = Nat.4 and Nat.0 = Nat.0 {
                        Nat.2 = Nat.4
                        Nat.0 = Nat.0
                        Nat.2 != Nat.4
                        false
                    } else {
                        false
                    }
                }
            }
        }
    }
    not c5_color(Nat.0, Nat.2)
}

/// The diagonal 0 -- 3 is blue.
theorem c5_diag_0_3 {
    not c5_color(Nat.0, Nat.3)
} by {
    if c5_color(Nat.0, Nat.3) {
        c5_color(Nat.0, Nat.3) = (Nat.0.suc = Nat.3 or Nat.3.suc = Nat.0 or (Nat.0 = Nat.4 and Nat.3 = Nat.0) or (Nat.3 = Nat.4 and Nat.0 = Nat.0))
        Nat.0.suc = Nat.1
        Nat.3.suc = Nat.4
        if Nat.0.suc = Nat.3 {
            Nat.0.suc = Nat.1
            Nat.1 = Nat.3
            Nat.1 != Nat.3
            false
        } else {
            if Nat.3.suc = Nat.0 {
                Nat.3.suc = Nat.4
                Nat.4 = Nat.0
                Nat.4 != Nat.0
                false
            } else {
                if Nat.0 = Nat.4 and Nat.3 = Nat.0 {
                    Nat.0 = Nat.4
                    Nat.3 = Nat.0
                    Nat.0 != Nat.4
                    false
                } else {
                    if Nat.3 = Nat.4 and Nat.0 = Nat.0 {
                        Nat.3 = Nat.4
                        Nat.0 = Nat.0
                        Nat.3 != Nat.4
                        false
                    } else {
                        false
                    }
                }
            }
        }
    }
    not c5_color(Nat.0, Nat.3)
}

/// The diagonal 1 -- 3 is blue.
theorem c5_diag_1_3 {
    not c5_color(Nat.1, Nat.3)
} by {
    if c5_color(Nat.1, Nat.3) {
        c5_color(Nat.1, Nat.3) = (Nat.1.suc = Nat.3 or Nat.3.suc = Nat.1 or (Nat.1 = Nat.4 and Nat.3 = Nat.0) or (Nat.3 = Nat.4 and Nat.1 = Nat.0))
        Nat.1.suc = Nat.2
        Nat.3.suc = Nat.4
        if Nat.1.suc = Nat.3 {
            Nat.1.suc = Nat.2
            Nat.2 = Nat.3
            Nat.2 != Nat.3
            false
        } else {
            if Nat.3.suc = Nat.1 {
                Nat.3.suc = Nat.4
                Nat.4 = Nat.1
                Nat.4 != Nat.1
                false
            } else {
                if Nat.1 = Nat.4 and Nat.3 = Nat.0 {
                    Nat.1 = Nat.4
                    Nat.3 = Nat.0
                    Nat.1 != Nat.4
                    false
                } else {
                    if Nat.3 = Nat.4 and Nat.1 = Nat.0 {
                        Nat.3 = Nat.4
                        Nat.1 = Nat.0
                        Nat.3 != Nat.4
                        false
                    } else {
                        false
                    }
                }
            }
        }
    }
    not c5_color(Nat.1, Nat.3)
}

/// The diagonal 1 -- 4 is blue.
theorem c5_diag_1_4 {
    not c5_color(Nat.1, Nat.4)
} by {
    if c5_color(Nat.1, Nat.4) {
        c5_color(Nat.1, Nat.4) = (Nat.1.suc = Nat.4 or Nat.4.suc = Nat.1 or (Nat.1 = Nat.4 and Nat.4 = Nat.0) or (Nat.4 = Nat.4 and Nat.1 = Nat.0))
        Nat.1.suc = Nat.2
        Nat.4.suc = Nat.5
        if Nat.1.suc = Nat.4 {
            Nat.1.suc = Nat.2
            Nat.2 = Nat.4
            Nat.2 != Nat.4
            false
        } else {
            if Nat.4.suc = Nat.1 {
                Nat.4.suc = Nat.5
                Nat.5 = Nat.1
                Nat.5 != Nat.1
                false
            } else {
                if Nat.1 = Nat.4 and Nat.4 = Nat.0 {
                    Nat.1 = Nat.4
                    Nat.4 = Nat.0
                    Nat.1 != Nat.4
                    false
                } else {
                    if Nat.4 = Nat.4 and Nat.1 = Nat.0 {
                        Nat.4 = Nat.4
                        Nat.1 = Nat.0
                        Nat.1 != Nat.0
                        false
                    } else {
                        false
                    }
                }
            }
        }
    }
    not c5_color(Nat.1, Nat.4)
}

/// The diagonal 2 -- 4 is blue.
theorem c5_diag_2_4 {
    not c5_color(Nat.2, Nat.4)
} by {
    if c5_color(Nat.2, Nat.4) {
        c5_color(Nat.2, Nat.4) = (Nat.2.suc = Nat.4 or Nat.4.suc = Nat.2 or (Nat.2 = Nat.4 and Nat.4 = Nat.0) or (Nat.4 = Nat.4 and Nat.2 = Nat.0))
        Nat.2.suc = Nat.3
        Nat.4.suc = Nat.5
        if Nat.2.suc = Nat.4 {
            Nat.2.suc = Nat.3
            Nat.3 = Nat.4
            Nat.3 != Nat.4
            false
        } else {
            if Nat.4.suc = Nat.2 {
                Nat.4.suc = Nat.5
                Nat.5 = Nat.2
                Nat.5 != Nat.2
                false
            } else {
                if Nat.2 = Nat.4 and Nat.4 = Nat.0 {
                    Nat.2 = Nat.4
                    Nat.4 = Nat.0
                    Nat.2 != Nat.4
                    false
                } else {
                    if Nat.4 = Nat.4 and Nat.2 = Nat.0 {
                        Nat.4 = Nat.4
                        Nat.2 = Nat.0
                        Nat.2 != Nat.0
                        false
                    } else {
                        false
                    }
                }
            }
        }
    }
    not c5_color(Nat.2, Nat.4)
}

/// The diagonal 2 -- 0 is blue.
theorem c5_diag_2_0 {
    not c5_color(Nat.2, Nat.0)
} by {
    if c5_color(Nat.2, Nat.0) {
        c5_color(Nat.2, Nat.0) = (Nat.2.suc = Nat.0 or Nat.0.suc = Nat.2 or (Nat.2 = Nat.4 and Nat.0 = Nat.0) or (Nat.0 = Nat.4 and Nat.2 = Nat.0))
        Nat.2.suc = Nat.3
        Nat.0.suc = Nat.1
        if Nat.2.suc = Nat.0 {
            Nat.2.suc = Nat.3
            Nat.3 = Nat.0
            Nat.3 != Nat.0
            false
        } else {
            if Nat.0.suc = Nat.2 {
                Nat.0.suc = Nat.1
                Nat.1 = Nat.2
                Nat.1 != Nat.2
                false
            } else {
                if Nat.2 = Nat.4 and Nat.0 = Nat.0 {
                    Nat.2 = Nat.4
                    Nat.0 = Nat.0
                    Nat.2 != Nat.4
                    false
                } else {
                    if Nat.0 = Nat.4 and Nat.2 = Nat.0 {
                        Nat.0 = Nat.4
                        Nat.2 = Nat.0
                        Nat.0 != Nat.4
                        false
                    } else {
                        false
                    }
                }
            }
        }
    }
    not c5_color(Nat.2, Nat.0)
}

/// The diagonal 3 -- 0 is blue.
theorem c5_diag_3_0 {
    not c5_color(Nat.3, Nat.0)
} by {
    if c5_color(Nat.3, Nat.0) {
        c5_color(Nat.3, Nat.0) = (Nat.3.suc = Nat.0 or Nat.0.suc = Nat.3 or (Nat.3 = Nat.4 and Nat.0 = Nat.0) or (Nat.0 = Nat.4 and Nat.3 = Nat.0))
        Nat.3.suc = Nat.4
        Nat.0.suc = Nat.1
        if Nat.3.suc = Nat.0 {
            Nat.3.suc = Nat.4
            Nat.4 = Nat.0
            Nat.4 != Nat.0
            false
        } else {
            if Nat.0.suc = Nat.3 {
                Nat.0.suc = Nat.1
                Nat.1 = Nat.3
                Nat.1 != Nat.3
                false
            } else {
                if Nat.3 = Nat.4 and Nat.0 = Nat.0 {
                    Nat.3 = Nat.4
                    Nat.0 = Nat.0
                    Nat.3 != Nat.4
                    false
                } else {
                    if Nat.0 = Nat.4 and Nat.3 = Nat.0 {
                        Nat.0 = Nat.4
                        Nat.3 = Nat.0
                        Nat.0 != Nat.4
                        false
                    } else {
                        false
                    }
                }
            }
        }
    }
    not c5_color(Nat.3, Nat.0)
}

/// The diagonal 3 -- 1 is blue.
theorem c5_diag_3_1 {
    not c5_color(Nat.3, Nat.1)
} by {
    if c5_color(Nat.3, Nat.1) {
        c5_color(Nat.3, Nat.1) = (Nat.3.suc = Nat.1 or Nat.1.suc = Nat.3 or (Nat.3 = Nat.4 and Nat.1 = Nat.0) or (Nat.1 = Nat.4 and Nat.3 = Nat.0))
        Nat.3.suc = Nat.4
        Nat.1.suc = Nat.2
        if Nat.3.suc = Nat.1 {
            Nat.3.suc = Nat.4
            Nat.4 = Nat.1
            Nat.4 != Nat.1
            false
        } else {
            if Nat.1.suc = Nat.3 {
                Nat.1.suc = Nat.2
                Nat.2 = Nat.3
                Nat.2 != Nat.3
                false
            } else {
                if Nat.3 = Nat.4 and Nat.1 = Nat.0 {
                    Nat.3 = Nat.4
                    Nat.1 = Nat.0
                    Nat.3 != Nat.4
                    false
                } else {
                    if Nat.1 = Nat.4 and Nat.3 = Nat.0 {
                        Nat.1 = Nat.4
                        Nat.3 = Nat.0
                        Nat.1 != Nat.4
                        false
                    } else {
                        false
                    }
                }
            }
        }
    }
    not c5_color(Nat.3, Nat.1)
}

/// The diagonal 4 -- 1 is blue.
theorem c5_diag_4_1 {
    not c5_color(Nat.4, Nat.1)
} by {
    if c5_color(Nat.4, Nat.1) {
        c5_color(Nat.4, Nat.1) = (Nat.4.suc = Nat.1 or Nat.1.suc = Nat.4 or (Nat.4 = Nat.4 and Nat.1 = Nat.0) or (Nat.1 = Nat.4 and Nat.4 = Nat.0))
        Nat.4.suc = Nat.5
        Nat.1.suc = Nat.2
        if Nat.4.suc = Nat.1 {
            Nat.4.suc = Nat.5
            Nat.5 = Nat.1
            Nat.5 != Nat.1
            false
        } else {
            if Nat.1.suc = Nat.4 {
                Nat.1.suc = Nat.2
                Nat.2 = Nat.4
                Nat.2 != Nat.4
                false
            } else {
                if Nat.4 = Nat.4 and Nat.1 = Nat.0 {
                    Nat.4 = Nat.4
                    Nat.1 = Nat.0
                    Nat.1 != Nat.0
                    false
                } else {
                    if Nat.1 = Nat.4 and Nat.4 = Nat.0 {
                        Nat.1 = Nat.4
                        Nat.4 = Nat.0
                        Nat.1 != Nat.4
                        false
                    } else {
                        false
                    }
                }
            }
        }
    }
    not c5_color(Nat.4, Nat.1)
}

/// The diagonal 4 -- 2 is blue.
theorem c5_diag_4_2 {
    not c5_color(Nat.4, Nat.2)
} by {
    if c5_color(Nat.4, Nat.2) {
        c5_color(Nat.4, Nat.2) = (Nat.4.suc = Nat.2 or Nat.2.suc = Nat.4 or (Nat.4 = Nat.4 and Nat.2 = Nat.0) or (Nat.2 = Nat.4 and Nat.4 = Nat.0))
        Nat.4.suc = Nat.5
        Nat.2.suc = Nat.3
        if Nat.4.suc = Nat.2 {
            Nat.4.suc = Nat.5
            Nat.5 = Nat.2
            Nat.5 != Nat.2
            false
        } else {
            if Nat.2.suc = Nat.4 {
                Nat.2.suc = Nat.3
                Nat.3 = Nat.4
                Nat.3 != Nat.4
                false
            } else {
                if Nat.4 = Nat.4 and Nat.2 = Nat.0 {
                    Nat.4 = Nat.4
                    Nat.2 = Nat.0
                    Nat.2 != Nat.0
                    false
                } else {
                    if Nat.2 = Nat.4 and Nat.4 = Nat.0 {
                        Nat.2 = Nat.4
                        Nat.4 = Nat.0
                        Nat.2 != Nat.4
                        false
                    } else {
                        false
                    }
                }
            }
        }
    }
    not c5_color(Nat.4, Nat.2)
}

/// No vertex is adjacent to itself in the C5 coloring.
theorem c5_loop_0_not {
    not c5_color(Nat.0, Nat.0)
} by {
    if c5_color(Nat.0, Nat.0) {
        c5_color(Nat.0, Nat.0) = (Nat.0.suc = Nat.0 or Nat.0.suc = Nat.0 or (Nat.0 = Nat.4 and Nat.0 = Nat.0) or (Nat.0 = Nat.4 and Nat.0 = Nat.0))
        if Nat.0.suc = Nat.0 {
            Nat.0.suc != Nat.0
            false
        } else {
            if Nat.0 = Nat.4 and Nat.0 = Nat.0 {
                Nat.0 = Nat.4
                Nat.0 != Nat.4
                false
            } else {
                false
            }
        }
    }
    not c5_color(Nat.0, Nat.0)
}

/// No vertex is adjacent to itself in the C5 coloring.
theorem c5_loop_1_not {
    not c5_color(Nat.1, Nat.1)
} by {
    if c5_color(Nat.1, Nat.1) {
        c5_color(Nat.1, Nat.1) = (Nat.1.suc = Nat.1 or Nat.1.suc = Nat.1 or (Nat.1 = Nat.4 and Nat.1 = Nat.0) or (Nat.1 = Nat.4 and Nat.1 = Nat.0))
        if Nat.1.suc = Nat.1 {
            Nat.1.suc != Nat.1
            false
        } else {
            if Nat.1 = Nat.4 and Nat.1 = Nat.0 {
                Nat.1 = Nat.4
                Nat.1 != Nat.4
                false
            } else {
                false
            }
        }
    }
    not c5_color(Nat.1, Nat.1)
}

/// No vertex is adjacent to itself in the C5 coloring.
theorem c5_loop_2_not {
    not c5_color(Nat.2, Nat.2)
} by {
    if c5_color(Nat.2, Nat.2) {
        c5_color(Nat.2, Nat.2) = (Nat.2.suc = Nat.2 or Nat.2.suc = Nat.2 or (Nat.2 = Nat.4 and Nat.2 = Nat.0) or (Nat.2 = Nat.4 and Nat.2 = Nat.0))
        if Nat.2.suc = Nat.2 {
            Nat.2.suc != Nat.2
            false
        } else {
            if Nat.2 = Nat.4 and Nat.2 = Nat.0 {
                Nat.2 = Nat.4
                Nat.2 != Nat.4
                false
            } else {
                false
            }
        }
    }
    not c5_color(Nat.2, Nat.2)
}

/// No vertex is adjacent to itself in the C5 coloring.
theorem c5_loop_3_not {
    not c5_color(Nat.3, Nat.3)
} by {
    if c5_color(Nat.3, Nat.3) {
        c5_color(Nat.3, Nat.3) = (Nat.3.suc = Nat.3 or Nat.3.suc = Nat.3 or (Nat.3 = Nat.4 and Nat.3 = Nat.0) or (Nat.3 = Nat.4 and Nat.3 = Nat.0))
        if Nat.3.suc = Nat.3 {
            Nat.3.suc != Nat.3
            false
        } else {
            if Nat.3 = Nat.4 and Nat.3 = Nat.0 {
                Nat.3 = Nat.4
                Nat.3 != Nat.4
                false
            } else {
                false
            }
        }
    }
    not c5_color(Nat.3, Nat.3)
}

/// No vertex is adjacent to itself in the C5 coloring.
theorem c5_loop_4_not {
    not c5_color(Nat.4, Nat.4)
} by {
    if c5_color(Nat.4, Nat.4) {
        c5_color(Nat.4, Nat.4) = (Nat.4.suc = Nat.4 or Nat.4.suc = Nat.4 or (Nat.4 = Nat.4 and Nat.4 = Nat.0) or (Nat.4 = Nat.4 and Nat.4 = Nat.0))
        if Nat.4.suc = Nat.4 {
            Nat.4.suc != Nat.4
            false
        } else {
            if Nat.4 = Nat.4 and Nat.4 = Nat.0 {
                Nat.4 = Nat.4
                Nat.4 != Nat.4
                false
            } else {
                false
            }
        }
    }
    not c5_color(Nat.4, Nat.4)
}

