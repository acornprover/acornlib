from nat import Nat, add_one_right, lte_and_lt, lt_and_lte, lt_imp_lte_suc, lte_trans,
    lt_or_lte, add_imp_sub_left, lte_antisymm
from finite_set import FiniteSet, fs_difference, fs_intersection,
    finite_set_difference_contains_eq, finite_set_difference_cardinality_is_sub_intersection,
    finite_set_intersection_cardinality_is_of_subset_right
from data.finite.finite_set_card import fs_card, fs_card_cardinality_is, fs_card_eq_of_cardinality_is
from graph.simple_graph_zero_forcing_closed import forcing_iterate_constant_of_closed
from data.finite.finite_set_card_difference import fs_card_difference_of_subset
from data.finite.finite_set_card_members import fs_two_distinct_members
from graph.simple_graph import SimpleGraph, complete_graph, complete_graph_adj_iff_ne
from graph.simple_graph_zero_forcing_rule import is_white_neighbor, can_force, can_force_apply,
    can_force_unique, is_forceable, is_forceable_witness, is_forcing_closed,
    is_forcing_closed_intro, derived_set
from graph.simple_graph_zero_forcing_closure import forcing_iterate, forcing_iterate_zero,
    forcing_iterate_suc, forcing_iterate_stationary, is_zero_forcing_set,
    is_zero_forcing_set_apply

numerals Nat

/// Two white vertices leave every blue vertex unable to force.
///
/// In the complete graph a blue vertex is adjacent to both of them, so neither is its only
/// white neighbour and the uniqueness clause of the rule fails.
theorem complete_graph_two_white_blocks_forcing[V](
    s: FiniteSet[V], b: FiniteSet[V], w1: V, w2: V
) {
    s.contains(w1) and not b.contains(w1) and s.contains(w2) and not b.contains(w2)
        and w1 != w2
        implies is_forcing_closed(complete_graph[V], s, b)
} by {
    if s.contains(w1) and not b.contains(w1) and s.contains(w2) and not b.contains(w2)
        and w1 != w2 {
        forall(v: V) {
            if is_forceable(complete_graph[V], s, b, v) {
                is_forceable_witness(complete_graph[V], s, b, v)
                let (u: V) satisfy {
                    can_force(complete_graph[V], s, b, u, v)
                }
                can_force_apply(complete_graph[V], s, b, u, v)
                b.contains(u)
                if u = w1 {
                    b.contains(w1)
                    false
                }
                u != w1
                if u = w2 {
                    b.contains(w2)
                    false
                }
                u != w2
                complete_graph_adj_iff_ne(u, w1)
                complete_graph[V].adj(u, w1) = (u != w1)
                complete_graph[V].adj(u, w1)
                (is_white_neighbor(complete_graph[V], s, b, u, w1)
                    = (s.contains(w1) and not b.contains(w1)
                        and complete_graph[V].adj(u, w1)))
                is_white_neighbor(complete_graph[V], s, b, u, w1)
                can_force_unique(complete_graph[V], s, b, u, v, w1)
                w1 = v
                complete_graph_adj_iff_ne(u, w2)
                complete_graph[V].adj(u, w2) = (u != w2)
                complete_graph[V].adj(u, w2)
                (is_white_neighbor(complete_graph[V], s, b, u, w2)
                    = (s.contains(w2) and not b.contains(w2)
                        and complete_graph[V].adj(u, w2)))
                is_white_neighbor(complete_graph[V], s, b, u, w2)
                can_force_unique(complete_graph[V], s, b, u, v, w2)
                w2 = v
                w1 = w2
                false
            }
            not is_forceable(complete_graph[V], s, b, v)
        }
        is_forcing_closed_intro(complete_graph[V], s, b)
        is_forcing_closed(complete_graph[V], s, b)
    }
}

/// A zero forcing set of the complete graph leaves at most one vertex uncoloured.
///
/// With two white vertices nothing can ever force, so the colouring is frozen at the start
/// and never reaches the whole set.
theorem complete_graph_zero_forcing_set_card[V](s: FiniteSet[V], b: FiniteSet[V]) {
    b.subset_eq(s) and is_zero_forcing_set(complete_graph[V], s, b)
        implies fs_card(fs_difference(s, b)) < Nat.2
} by {
    if b.subset_eq(s) and is_zero_forcing_set(complete_graph[V], s, b) {
        if Nat.2 <= fs_card(fs_difference(s, b)) {
            fs_two_distinct_members(fs_difference(s, b))
            let (w1: V, w2: V) satisfy {
                fs_difference(s, b).contains(w1) and fs_difference(s, b).contains(w2)
                    and w1 != w2
            }
            finite_set_difference_contains_eq(s, b, w1)
            s.contains(w1) and not b.contains(w1)
            finite_set_difference_contains_eq(s, b, w2)
            s.contains(w2) and not b.contains(w2)
            complete_graph_two_white_blocks_forcing(s, b, w1, w2)
            is_forcing_closed(complete_graph[V], s, b)
            is_zero_forcing_set_apply(complete_graph[V], s, b)
            exists(n: Nat) { forcing_iterate(complete_graph[V], s, b, n) = s }
            let (n: Nat) satisfy {
                forcing_iterate(complete_graph[V], s, b, n) = s
            }
            forcing_iterate_constant_of_closed(complete_graph[V], s, b, n)
            forcing_iterate(complete_graph[V], s, b, n) = b
            b = s
            finite_set_difference_contains_eq(s, b, w1)
            not b.contains(w1)
            not s.contains(w1)
            false
        }
        fs_card(fs_difference(s, b)) < Nat.2
    }
}
