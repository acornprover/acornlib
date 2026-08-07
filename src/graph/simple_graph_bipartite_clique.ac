from nat import Nat
from finite_set import FiniteSet, finite_set_subset_contains
from graph.simple_graph import SimpleGraph
from graph.simple_graph_bipartite import is_bipartition
from graph.simple_graph_independent import is_clique_in, is_clique_in_apply
from graph.simple_graph_triangle_free import is_triangle_free, is_triangle_free_no_triangle
from graph.simple_graph_triangle_consequences import bipartition_is_triangle_free

numerals Nat

/// A triangle-free graph has no clique on three distinct vertices.
///
/// Three distinct members of a clique are mutually adjacent, which is exactly the
/// triangle a triangle-free graph forbids.
theorem triangle_free_clique_no_three[V](
    g: SimpleGraph[V], s: FiniteSet[V], t: FiniteSet[V], x: V, y: V, z: V
) {
    is_triangle_free(g, s) and t.subset_eq(s) and is_clique_in(g, t) and t.contains(x) and t.contains(y) and t.contains(z) and x != y and x != z implies y = z
} by {
    if is_triangle_free(g, s) and t.subset_eq(s) and is_clique_in(g, t) and t.contains(x) and t.contains(y) and t.contains(z) and x != y and x != z {
        finite_set_subset_contains(t, s, x)
        finite_set_subset_contains(t, s, y)
        finite_set_subset_contains(t, s, z)
        s.contains(x)
        s.contains(y)
        s.contains(z)
        is_clique_in_apply(g, t, x, y)
        g.adj(x, y)
        is_clique_in_apply(g, t, x, z)
        g.adj(x, z)
        is_triangle_free_no_triangle(g, s, x, y, z)
        not g.adj(y, z)
        if y != z {
            is_clique_in_apply(g, t, y, z)
            g.adj(y, z)
            false
        }
        y = z
    }
}

/// A bipartite graph has no clique on three distinct vertices.
///
/// Bipartite graphs are triangle-free, and a three-vertex clique is a triangle.
theorem bipartite_clique_no_three[V](
    g: SimpleGraph[V], s: FiniteSet[V], t: FiniteSet[V], part: V -> Bool, x: V, y: V, z: V
) {
    is_bipartition(g, s, part) and t.subset_eq(s) and is_clique_in(g, t) and t.contains(x) and t.contains(y) and t.contains(z) and x != y and x != z implies y = z
} by {
    if is_bipartition(g, s, part) and t.subset_eq(s) and is_clique_in(g, t) and t.contains(x) and t.contains(y) and t.contains(z) and x != y and x != z {
        bipartition_is_triangle_free(g, s, part)
        is_triangle_free(g, s)
        triangle_free_clique_no_three(g, s, t, x, y, z)
        y = z
    }
}

/// In a triangle-free graph, a clique member has at most one other member.
///
/// The contrapositive reading of the three-vertex bound: fixing one member, every
/// other member is the same vertex.
theorem triangle_free_clique_second_unique[V](
    g: SimpleGraph[V], s: FiniteSet[V], t: FiniteSet[V], x: V, y: V, z: V
) {
    is_triangle_free(g, s) and t.subset_eq(s) and is_clique_in(g, t) and t.contains(x) and t.contains(y) and t.contains(z) and y != x and z != x implies y = z
} by {
    if is_triangle_free(g, s) and t.subset_eq(s) and is_clique_in(g, t) and t.contains(x) and t.contains(y) and t.contains(z) and y != x and z != x {
        x != y
        x != z
        triangle_free_clique_no_three(g, s, t, x, y, z)
        y = z
    }
}
