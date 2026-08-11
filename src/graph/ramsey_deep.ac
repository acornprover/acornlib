// ============================================================================
// Ramsey numbers, deepening: R(2, k) = k, R(3, 3) = 6, R(3, 4) <= 9,
// R(3, 5) <= 14, and the general binomial bound R(k, l) <= C(k + l - 2, k - 1).
//
// This file collects the classical small Ramsey numbers in one place, restating
// the library's verified results (graph/ramsey.ac for R(3, 3) <= 6,
// graph/ramsey34_upper.ac for the R(3, 4) <= 9 pigeonhole, and the arithmetic
// consequences recorded in graph/ramsey_numbers.ac) and adding the trivial
// case R(2, k) = k, which is proved in full generality here: with k vertices,
// either some edge is red (a red K2) or no edge is red, in which case the whole
// vertex set is a blue Kk.
//
// The classical ladder R(2, 5) + R(3, 4) = 5 + 9 = 14 for R(3, 5) <= 14 is
// restated below; the general bound R(k, l) <= C(k + l - 2, k - 1) is stated
// (with the two small instances R(2, 3) <= 3 and R(3, 3) <= 6 verified), and
// the full statements whose proofs are not yet verified in this environment
// are recorded as comments at the end, per the project convention.
// ============================================================================

from nat import Nat, not_lt_zero, lt_suc_right, lt_and_lte, suc_sub_one, add_zero_left,
    mul_cancel_left, add_suc_left
from finite_set import FiniteSet
from data.basic.logic import exists_intro
from data.nat.nat_range_set import range_set, range_set_contains, range_set_lt
from list import List, length_range, range_contains_iff_lt, range_is_unique,
    singleton_unique, singleton_contains_imp_eq
from data.list.list_cons_membership import cons_contains_cases, nil_not_contains
from graph.simple_graph import complete_graph, complete_graph_adj_iff_ne
from graph.ramsey import is_edge_2_coloring, edge_2_coloring_comm, has_red_triangle,
    has_blue_triangle, has_mono_triangle, has_red_triangle_intro, has_blue_triangle_intro,
    ramsey_r33_upper, six_vertices
from graph.ramsey34_vertices import nine_vertices
from graph.ramsey34_k4 import has_red_k4, has_blue_k4, has_mono_k4, has_red_k4_intro,
    has_blue_k4_intro
from graph.ramsey34_pigeonhole import four_edges_from_zero_share_color,
    four_edges_from_zero_share_color_pigeonhole, four_edges_from_zero_share_color_witness
from graph.ramsey34_upper import ramsey_r34_upper
from graph.ramsey_numbers import ramsey_number_upper, has_red_clique, has_blue_clique,
    clique_members, clique_pairwise_red, clique_pairwise_blue, blue_triangle_to_clique3,
    ramsey_r33_upper_number, ramsey_r35_bound_arith, ramsey_r35_from_recurrence,
    ramsey_recurrence, get_idx_some_of_lt, is_unique_distinct_values, value_at_imp_contains
from graph.ramsey_numbers_esz import value_at
from combinatorics import binom, choose_one, binom_two_double

numerals Nat

// ============================================================================
// Section A: R(2, k) = k.
//
// With k vertices, either some edge is red (a red clique of size two, witnessed
// by its two endpoints) or no edge is red, in which case every pair of vertices
// is blue and the whole vertex set is a blue clique of size k.  The upper bound
// R(2, k) <= k is proved for every k below; the lower bound R(2, k) > k - 1
// (a constantly blue coloring of K_{k - 1} has no red edge and, having only
// k - 1 vertices, no blue Kk) is verified for k = 3, which settles R(2, 3) = 3.
// ============================================================================

/// The length of the two-element witness list is two.
theorem two_list_length[T](x: T, y: T) {
    List.cons(x, List.cons(y, List.nil[T])).length = Nat.2
} by {
    List.nil[T].length = Nat.0
    List.cons(y, List.nil[T]).length = List.nil[T].length.suc
    List.cons(y, List.nil[T]).length = Nat.0.suc
    Nat.0.suc = Nat.1
    List.cons(y, List.nil[T]).length = Nat.1
    List.cons(x, List.cons(y, List.nil[T])).length = List.cons(y, List.nil[T]).length.suc
    List.cons(x, List.cons(y, List.nil[T])).length = Nat.1.suc
    Nat.1.suc = Nat.2
    List.cons(x, List.cons(y, List.nil[T])).length = Nat.2
}

/// Two distinct elements form a unique two-element list.
theorem two_list_unique[T](x: T, y: T) {
    x != y implies List.cons(x, List.cons(y, List.nil[T])).is_unique
} by {
    if x != y {
        List.nil[T].unique = List.nil[T]
        List.nil[T].contains(y) = false
        List.cons(y, List.nil[T]).unique = List.cons(y, List.nil[T].unique)
        List.cons(y, List.nil[T]).unique = List.cons(y, List.nil[T])
        if List.singleton(y).contains(x) {
            singleton_contains_imp_eq(y, x)
            x = y
            false
        }
        not List.singleton(y).contains(x)
        List.cons(x, List.singleton(y)).unique = List.cons(x, List.singleton(y).unique)
        List.cons(x, List.singleton(y)).unique = List.cons(x, List.cons(y, List.nil[T]))
        List.cons(x, List.cons(y, List.nil[T])).unique = List.cons(x, List.cons(y, List.nil[T]))
        List.cons(x, List.cons(y, List.nil[T])).is_unique
    }
}

/// A member of the two-element witness list is one of its two elements.
theorem two_list_contains_cases[T](x: T, y: T, a: T) {
    List.cons(x, List.cons(y, List.nil[T])).contains(a)
        implies a = x or a = y
} by {
    if List.cons(x, List.cons(y, List.nil[T])).contains(a) {
        cons_contains_cases(x, List.cons(y, List.nil[T]), a)
        a = x or List.cons(y, List.nil[T]).contains(a)
        if a = x {
            a = x or a = y
        } else {
            List.cons(y, List.nil[T]).contains(a)
            cons_contains_cases(y, List.nil[T], a)
            a = y or List.nil[T].contains(a)
            if a = y {
                a = x or a = y
            } else {
                List.nil[T].contains(a)
                nil_not_contains(a)
                not List.nil[T].contains(a)
                false
            }
            a = x or a = y
        }
        a = x or a = y
    }
}

/// Two elements with a red edge between them: every two members of the pair are
/// the same element or red-joined (using symmetry of the coloring for the
/// reversed pair).
theorem red_pairwise_two[V](color: (V, V) -> Bool, x: V, y: V, a: V, b: V) {
    is_edge_2_coloring(complete_graph[V], color) and color(x, y) and
    (a = x or a = y) and (b = x or b = y)
    implies (a = b or color(a, b))
} by {
    if is_edge_2_coloring(complete_graph[V], color) and color(x, y) and
        (a = x or a = y) and (b = x or b = y) {
        edge_2_coloring_comm(complete_graph[V], color, x, y)
        complete_graph_adj_iff_ne[V](x, y)
        complete_graph[V].adj(x, y) = (x != y)
        if a = x {
            if b = x {
                a = b
                a = b or color(a, b)
            } else {
                b = y
                color(x, y)
                color(a, b)
                a = b or color(a, b)
            }
            a = b or color(a, b)
        } else {
            a = y
            if b = x {
                x != y
                complete_graph[V].adj(x, y)
                edge_2_coloring_comm(complete_graph[V], color, x, y)
                color(x, y) = color(y, x)
                color(y, x)
                color(a, b)
                a = b or color(a, b)
            } else {
                b = y
                a = b
                a = b or color(a, b)
            }
            a = b or color(a, b)
        }
        a = b or color(a, b)
    }
}

/// A red edge in `s` is a red clique of size two in `s`.
theorem red_clique2_of_edge[V](color: (V, V) -> Bool, s: FiniteSet[V], x: V, y: V) {
    is_edge_2_coloring(complete_graph[V], color) and
    s.contains(x) and s.contains(y) and x != y and color(x, y)
    implies has_red_clique(color, s, Nat.2)
} by {
    if is_edge_2_coloring(complete_graph[V], color) and
        s.contains(x) and s.contains(y) and x != y and color(x, y) {
        two_list_length(x, y)
        List.cons(x, List.cons(y, List.nil[V])).length = Nat.2
        two_list_unique(x, y)
        List.cons(x, List.cons(y, List.nil[V])).is_unique
        forall(u: V) {
            if List.cons(x, List.cons(y, List.nil[V])).contains(u) {
                two_list_contains_cases(x, y, u)
                u = x or u = y
                if u = x {
                    s.contains(u)
                } else {
                    u = y
                    s.contains(u)
                }
                s.contains(u)
            }
        }
        forall(u: V) {
            List.cons(x, List.cons(y, List.nil[V])).contains(u) implies s.contains(u)
        }
        forall(u: V, v: V) {
            if List.cons(x, List.cons(y, List.nil[V])).contains(u)
                and List.cons(x, List.cons(y, List.nil[V])).contains(v) {
                two_list_contains_cases(x, y, u)
                u = x or u = y
                two_list_contains_cases(x, y, v)
                v = x or v = y
                edge_2_coloring_comm(complete_graph[V], color, x, y)
                complete_graph_adj_iff_ne[V](x, y)
                complete_graph[V].adj(x, y) = (x != y)
                if u = x {
                    if v = x {
                        u = v
                        u = v or color(u, v)
                    } else {
                        v = y
                        color(x, y)
                        color(u, v)
                        u = v or color(u, v)
                    }
                    u = v or color(u, v)
                } else {
                    u = y
                    if v = x {
                        x != y
                        complete_graph[V].adj(x, y)
                        edge_2_coloring_comm(complete_graph[V], color, x, y)
                        color(x, y) = color(y, x)
                        color(y, x)
                        color(u, v)
                        u = v or color(u, v)
                    } else {
                        v = y
                        u = v
                        u = v or color(u, v)
                    }
                    u = v or color(u, v)
                }
                u = v or color(u, v)
            }
        }
        forall(u: V, v: V) {
            List.cons(x, List.cons(y, List.nil[V])).contains(u) and List.cons(x, List.cons(y, List.nil[V])).contains(v) implies (u = v or color(u, v))
        }
        has_red_clique(color, s, Nat.2) = exists(xs: List[V]) {
            xs.length = Nat.2 and xs.is_unique and clique_members(xs, s) and clique_pairwise_red(color, xs)
        }
        clique_members(List.cons(x, List.cons(y, List.nil[V])), s)
        clique_pairwise_red(color, List.cons(x, List.cons(y, List.nil[V])))
        (List.cons(x, List.cons(y, List.nil[V])).length = Nat.2
            and List.cons(x, List.cons(y, List.nil[V])).is_unique
            and clique_members(List.cons(x, List.cons(y, List.nil[V])), s)
            and clique_pairwise_red(color, List.cons(x, List.cons(y, List.nil[V]))))
        exists_intro(function(xs: List[V]) {
            xs.length = Nat.2 and xs.is_unique and clique_members(xs, s) and clique_pairwise_red(color, xs)
        }, List.cons(x, List.cons(y, List.nil[V])))
        exists(xs: List[V]) {
            xs.length = Nat.2 and xs.is_unique and clique_members(xs, s) and clique_pairwise_red(color, xs)
        }
        has_red_clique(color, s, Nat.2)
    }
}

/// R(2, k) <= k: every red/blue 2-coloring of the complete graph on k vertices
/// has a red edge or a blue clique of size k.
///
/// The vertex list `k.range` lists all k vertices.  Either some edge of the
/// coloring is red, which is a red K2 (`red_clique2_of_edge`), or no edge is
/// red, in which case `k.range` is a blue Kk: its members are exactly the
/// vertices of `range_set(k)`, and any two distinct members are blue-joined
/// (a red join would be a red K2).
theorem ramsey_r2k_upper(k: Nat) {
    ramsey_number_upper(Nat.2, k, k)
} by {
    ramsey_number_upper(Nat.2, k, k) = forall(color: (Nat, Nat) -> Bool) {
        is_edge_2_coloring(complete_graph[Nat], color) implies (has_red_clique(color, range_set(k), Nat.2) or has_blue_clique(color, range_set(k), k))
    }
    forall(color: (Nat, Nat) -> Bool) {
        if is_edge_2_coloring(complete_graph[Nat], color) {
            if has_red_clique(color, range_set(k), Nat.2) {
                has_red_clique(color, range_set(k), Nat.2) or has_blue_clique(color, range_set(k), k)
            } else {
                length_range(k)
                k.range.length = k
                range_is_unique(k)
                k.range.is_unique
                forall(x: Nat) {
                    if k.range.contains(x) {
                        range_contains_iff_lt(k, x)
                        k.range.contains(x) = (x < k)
                        x < k
                        range_set_contains(k, x)
                        range_set(k).contains(x)
                    }
                }
                forall(x: Nat) {
                    k.range.contains(x) implies range_set(k).contains(x)
                }
                forall(x: Nat, y: Nat) {
                    if k.range.contains(x) and k.range.contains(y) {
                        range_contains_iff_lt(k, x)
                        k.range.contains(x) = (x < k)
                        x < k
                        range_set_contains(k, x)
                        range_set(k).contains(x)
                        range_contains_iff_lt(k, y)
                        k.range.contains(y) = (y < k)
                        y < k
                        range_set_contains(k, y)
                        range_set(k).contains(y)
                        if x = y {
                            x = y or not color(x, y)
                        } else {
                            if color(x, y) {
                                red_clique2_of_edge(color, range_set(k), x, y)
                                has_red_clique(color, range_set(k), Nat.2)
                                false
                            }
                            not color(x, y)
                            x = y or not color(x, y)
                        }
                    }
                }
                forall(x: Nat, y: Nat) {
                    k.range.contains(x) and k.range.contains(y) implies (x = y or not color(x, y))
                }
                has_blue_clique(color, range_set(k), k) = exists(xs: List[Nat]) {
                    xs.length = k and xs.is_unique and clique_members(xs, range_set(k)) and clique_pairwise_blue(color, xs)
                }
                clique_members(k.range, range_set(k))
                clique_pairwise_blue(color, k.range)
                (k.range.length = k and k.range.is_unique
                    and clique_members(k.range, range_set(k))
                    and clique_pairwise_blue(color, k.range))
                exists_intro(function(xs: List[Nat]) {
                    xs.length = k and xs.is_unique and clique_members(xs, range_set(k)) and clique_pairwise_blue(color, xs)
                }, k.range)
                exists(xs: List[Nat]) {
                    xs.length = k and xs.is_unique and clique_members(xs, range_set(k)) and clique_pairwise_blue(color, xs)
                }
                has_blue_clique(color, range_set(k), k)
                has_red_clique(color, range_set(k), Nat.2) or has_blue_clique(color, range_set(k), k)
            }
        }
    }
    ramsey_number_upper(Nat.2, k, k)
}

/// R(2, 3) <= 3: the case k = 3 of the general upper bound.
theorem ramsey_r23_upper {
    ramsey_number_upper(Nat.2, Nat.3, Nat.3)
} by {
    ramsey_r2k_upper(Nat.3)
    ramsey_number_upper(Nat.2, Nat.3, Nat.3)
}

// ----------------------------------------------------------------------------
// The lower bound R(2, 3) > 2.
//
// The complete graph on two vertices, colored constantly blue, has no red edge
// (hence no red K2) and, having only two vertices, no blue K3.
// ----------------------------------------------------------------------------

/// The lower-bound coloring for R(2, 3) > 2: every edge is blue.
define r23_lb_color(x: Nat, y: Nat) -> Bool {
    false
}

/// The constantly-blue coloring is an edge 2-coloring of the complete graph.
theorem r23_lb_color_is_edge_2_coloring {
    is_edge_2_coloring(complete_graph[Nat], r23_lb_color)
} by {
    forall(x: Nat, y: Nat) {
        if complete_graph[Nat].adj(x, y) {
            r23_lb_color(x, y) = false
            r23_lb_color(y, x) = false
            false = false
            r23_lb_color(x, y) = r23_lb_color(y, x)
        }
        complete_graph[Nat].adj(x, y) implies r23_lb_color(x, y) = r23_lb_color(y, x)
    }
    is_edge_2_coloring(complete_graph[Nat], r23_lb_color) = forall(a: Nat, b: Nat) {
        complete_graph[Nat].adj(a, b) implies r23_lb_color(a, b) = r23_lb_color(b, a)
    }
    is_edge_2_coloring(complete_graph[Nat], r23_lb_color)
}

/// The constantly-blue coloring of the two vertices has no red clique of size two.
///
/// A red K2 would be a unique two-element list of vertices whose two members
/// are red-joined; but every edge is blue, so the two distinct members of the
/// list would have to be equal.
theorem r23_lb_no_red_clique {
    not has_red_clique(r23_lb_color, range_set(Nat.2), Nat.2)
} by {
    if has_red_clique(r23_lb_color, range_set(Nat.2), Nat.2) {
        has_red_clique(r23_lb_color, range_set(Nat.2), Nat.2) = exists(xs: List[Nat]) {
            xs.length = Nat.2 and xs.is_unique and clique_members(xs, range_set(Nat.2)) and clique_pairwise_red(r23_lb_color, xs)
        }
        let (xs: List[Nat]) satisfy {
            xs.length = Nat.2 and xs.is_unique and clique_members(xs, range_set(Nat.2)) and clique_pairwise_red(r23_lb_color, xs)
        }
        xs.length = Nat.2
        Nat.0.suc = Nat.1
        Nat.0 < Nat.0.suc
        Nat.0 < Nat.1
        Nat.1.suc = Nat.2
        Nat.1 < Nat.1.suc
        Nat.1 < Nat.2
        lt_and_lte(Nat.0, Nat.1, Nat.2)
        Nat.0 < Nat.2
        xs.length = Nat.2
        Nat.0 < xs.length
        get_idx_some_of_lt(xs, Nat.0)
        exists(x: Nat) { xs.get_idx(Nat.0) = Option.some(x) }
        let (x: Nat) satisfy { xs.get_idx(Nat.0) = Option.some(x) }
        xs.length = Nat.2
        Nat.1 < xs.length
        get_idx_some_of_lt(xs, Nat.1)
        exists(y: Nat) { xs.get_idx(Nat.1) = Option.some(y) }
        let (y: Nat) satisfy { xs.get_idx(Nat.1) = Option.some(y) }
        value_at(xs, Nat.0, x)
        value_at(xs, Nat.1, y)
        xs.is_unique
        Nat.0 != Nat.1
        is_unique_distinct_values(xs, Nat.0, Nat.1)
        not exists(z: Nat) { value_at(xs, Nat.0, z) and value_at(xs, Nat.1, z) }
        if x = y {
            value_at(xs, Nat.0, x) and value_at(xs, Nat.1, x)
            exists(z: Nat) { value_at(xs, Nat.0, z) and value_at(xs, Nat.1, z) }
            false
        }
        x != y
        value_at_imp_contains(xs, Nat.0, x)
        xs.contains(x)
        value_at_imp_contains(xs, Nat.1, y)
        xs.contains(y)
        clique_pairwise_red(r23_lb_color, xs) = forall(u: Nat, v: Nat) {
            xs.contains(u) and xs.contains(v) implies (u = v or r23_lb_color(u, v))
        }
        forall(u: Nat, v: Nat) {
            xs.contains(u) and xs.contains(v) implies (u = v or r23_lb_color(u, v))
        }
        xs.contains(x) and xs.contains(y) implies (x = y or r23_lb_color(x, y))
        xs.contains(x) and xs.contains(y)
        x = y or r23_lb_color(x, y)
        r23_lb_color(x, y) = false
        not r23_lb_color(x, y)
        x = y
        x != y
        false
    }
    not has_red_clique(r23_lb_color, range_set(Nat.2), Nat.2)
}

/// A member of the two-element range set is one of its two labels.
theorem lt_two_cases(x: Nat) {
    x < Nat.2 implies (x = Nat.0 or x = Nat.1)
} by {
    if x < Nat.2 {
        lt_suc_right(x, Nat.1)
        (x = Nat.1 or x < Nat.1)
        if x = Nat.1 {
            x = Nat.0 or x = Nat.1
        } else {
            x < Nat.1
            lt_suc_right(x, Nat.0)
            (x = Nat.0 or x < Nat.0)
            if x = Nat.0 {
                x = Nat.0 or x = Nat.1
            } else {
                x < Nat.0
                not_lt_zero(x)
                not (x < Nat.0)
                false
            }
            x = Nat.0 or x = Nat.1
        }
        x = Nat.0 or x = Nat.1
    }
}

/// Three naturals below two are not pairwise distinct: two of them are equal.
theorem three_lt_two_not_distinct(x: Nat, y: Nat, z: Nat) {
    x < Nat.2 and y < Nat.2 and z < Nat.2 implies (x = y or x = z or y = z)
} by {
    if x < Nat.2 and y < Nat.2 and z < Nat.2 {
        lt_two_cases(x)
        x = Nat.0 or x = Nat.1
        lt_two_cases(y)
        y = Nat.0 or y = Nat.1
        lt_two_cases(z)
        z = Nat.0 or z = Nat.1
        if x = Nat.0 {
            if y = Nat.0 {
                x = y
                x = y or x = z or y = z
            } else {
                y != Nat.0
                y = Nat.0 or y = Nat.1
                y = Nat.1
                if z = Nat.0 {
                    x = z
                    x = y or x = z or y = z
                } else {
                    z != Nat.0
                    z = Nat.0 or z = Nat.1
                    z = Nat.1
                    y = z
                    x = y or x = z or y = z
                }
                x = y or x = z or y = z
            }
            x = y or x = z or y = z
        } else {
            x != Nat.0
            x = Nat.0 or x = Nat.1
            x = Nat.1
            if y = Nat.1 {
                x = y
                x = y or x = z or y = z
            } else {
                y != Nat.1
                y = Nat.0 or y = Nat.1
                y = Nat.0
                if z = Nat.1 {
                    x = z
                    x = y or x = z or y = z
                } else {
                    z != Nat.1
                    z = Nat.0 or z = Nat.1
                    z = Nat.0
                    y = z
                    x = y or x = z or y = z
                }
                x = y or x = z or y = z
            }
            x = y or x = z or y = z
        }
        x = y or x = z or y = z
    }
}

/// The constantly-blue coloring of the two vertices has no blue clique of size three.
///
/// A blue K3 would be a unique three-element list of vertices of the two-element
/// set, hence three pairwise distinct naturals below two, which do not exist.
theorem r23_lb_no_blue_clique {
    not has_blue_clique(r23_lb_color, range_set(Nat.2), Nat.3)
} by {
    if has_blue_clique(r23_lb_color, range_set(Nat.2), Nat.3) {
        has_blue_clique(r23_lb_color, range_set(Nat.2), Nat.3) = exists(xs: List[Nat]) {
            xs.length = Nat.3 and xs.is_unique and clique_members(xs, range_set(Nat.2)) and clique_pairwise_blue(r23_lb_color, xs)
        }
        let (xs: List[Nat]) satisfy {
            xs.length = Nat.3 and xs.is_unique and clique_members(xs, range_set(Nat.2)) and clique_pairwise_blue(r23_lb_color, xs)
        }
        xs.length = Nat.3
        Nat.0.suc = Nat.1
        Nat.0 < Nat.0.suc
        Nat.0 < Nat.1
        Nat.1.suc = Nat.2
        Nat.1 < Nat.1.suc
        Nat.1 < Nat.2
        Nat.2.suc = Nat.3
        Nat.2 < Nat.2.suc
        Nat.2 < Nat.3
        lt_and_lte(Nat.0, Nat.1, Nat.3)
        Nat.0 < Nat.3
        xs.length = Nat.3
        Nat.0 < xs.length
        get_idx_some_of_lt(xs, Nat.0)
        exists(x: Nat) { xs.get_idx(Nat.0) = Option.some(x) }
        let (x: Nat) satisfy { xs.get_idx(Nat.0) = Option.some(x) }
        Nat.1 < Nat.2
        Nat.2 < Nat.3
        Nat.2 <= Nat.3
        lt_and_lte(Nat.1, Nat.2, Nat.3)
        Nat.1 < Nat.3
        xs.length = Nat.3
        Nat.1 < xs.length
        get_idx_some_of_lt(xs, Nat.1)
        exists(y: Nat) { xs.get_idx(Nat.1) = Option.some(y) }
        let (y: Nat) satisfy { xs.get_idx(Nat.1) = Option.some(y) }
        xs.length = Nat.3
        Nat.2 < xs.length
        get_idx_some_of_lt(xs, Nat.2)
        exists(z: Nat) { xs.get_idx(Nat.2) = Option.some(z) }
        let (z: Nat) satisfy { xs.get_idx(Nat.2) = Option.some(z) }
        value_at(xs, Nat.0, x)
        value_at(xs, Nat.1, y)
        value_at(xs, Nat.2, z)
        xs.is_unique
        Nat.0 != Nat.1
        is_unique_distinct_values(xs, Nat.0, Nat.1)
        not exists(a: Nat) { value_at(xs, Nat.0, a) and value_at(xs, Nat.1, a) }
        if x = y {
            value_at(xs, Nat.0, x) and value_at(xs, Nat.1, x)
            exists(a: Nat) { value_at(xs, Nat.0, a) and value_at(xs, Nat.1, a) }
            false
        }
        x != y
        Nat.0 != Nat.2
        is_unique_distinct_values(xs, Nat.0, Nat.2)
        not exists(a: Nat) { value_at(xs, Nat.0, a) and value_at(xs, Nat.2, a) }
        if x = z {
            value_at(xs, Nat.0, x) and value_at(xs, Nat.2, x)
            exists(a: Nat) { value_at(xs, Nat.0, a) and value_at(xs, Nat.2, a) }
            false
        }
        x != z
        Nat.1 != Nat.2
        is_unique_distinct_values(xs, Nat.1, Nat.2)
        not exists(a: Nat) { value_at(xs, Nat.1, a) and value_at(xs, Nat.2, a) }
        if y = z {
            value_at(xs, Nat.1, y) and value_at(xs, Nat.2, y)
            exists(a: Nat) { value_at(xs, Nat.1, a) and value_at(xs, Nat.2, a) }
            false
        }
        y != z
        value_at_imp_contains(xs, Nat.0, x)
        xs.contains(x)
        clique_members(xs, range_set(Nat.2)) = forall(a: Nat) {
            xs.contains(a) implies range_set(Nat.2).contains(a)
        }
        forall(a: Nat) {
            xs.contains(a) implies range_set(Nat.2).contains(a)
        }
        xs.contains(x) implies range_set(Nat.2).contains(x)
        xs.contains(x)
        range_set(Nat.2).contains(x)
        range_set_lt(Nat.2, x)
        range_set(Nat.2).contains(x) implies x < Nat.2
        x < Nat.2
        value_at_imp_contains(xs, Nat.1, y)
        xs.contains(y)
        xs.contains(y) implies range_set(Nat.2).contains(y)
        xs.contains(y)
        range_set(Nat.2).contains(y)
        range_set_lt(Nat.2, y)
        y < Nat.2
        value_at_imp_contains(xs, Nat.2, z)
        xs.contains(z)
        xs.contains(z) implies range_set(Nat.2).contains(z)
        xs.contains(z)
        range_set(Nat.2).contains(z)
        range_set_lt(Nat.2, z)
        z < Nat.2
        x < Nat.2 and y < Nat.2 and z < Nat.2
        three_lt_two_not_distinct(x, y, z)
        x = y or x = z or y = z
        x != z
        y != z
        x = y
        x != y
        false
    }
    not has_blue_clique(r23_lb_color, range_set(Nat.2), Nat.3)
}

/// R(2, 3) > 2: the complete graph on two vertices has a coloring with no red
/// edge and no blue triangle.
theorem ramsey_r23_lower_bound {
    exists(color: (Nat, Nat) -> Bool) {
        is_edge_2_coloring(complete_graph[Nat], color) and
        not has_red_clique(color, range_set(Nat.2), Nat.2) and
        not has_blue_clique(color, range_set(Nat.2), Nat.3)
    }
} by {
    r23_lb_color_is_edge_2_coloring
    is_edge_2_coloring(complete_graph[Nat], r23_lb_color)
    r23_lb_no_red_clique
    not has_red_clique(r23_lb_color, range_set(Nat.2), Nat.2)
    r23_lb_no_blue_clique
    not has_blue_clique(r23_lb_color, range_set(Nat.2), Nat.3)
    is_edge_2_coloring(complete_graph[Nat], r23_lb_color) and
        not has_red_clique(r23_lb_color, range_set(Nat.2), Nat.2) and
        not has_blue_clique(r23_lb_color, range_set(Nat.2), Nat.3)
    exists_intro(function(color: (Nat, Nat) -> Bool) {
        is_edge_2_coloring(complete_graph[Nat], color) and
        not has_red_clique(color, range_set(Nat.2), Nat.2) and
        not has_blue_clique(color, range_set(Nat.2), Nat.3)
    }, r23_lb_color)
    exists(color: (Nat, Nat) -> Bool) {
        is_edge_2_coloring(complete_graph[Nat], color) and
        not has_red_clique(color, range_set(Nat.2), Nat.2) and
        not has_blue_clique(color, range_set(Nat.2), Nat.3)
    }
}

/// R(2, 3) = 3: the upper bound R(2, 3) <= 3 and the lower bound R(2, 3) > 2.
theorem ramsey_r23_exact {
    ramsey_number_upper(Nat.2, Nat.3, Nat.3) and
    exists(color: (Nat, Nat) -> Bool) {
        is_edge_2_coloring(complete_graph[Nat], color) and
        not has_red_clique(color, range_set(Nat.2), Nat.2) and
        not has_blue_clique(color, range_set(Nat.2), Nat.3)
    }
} by {
    ramsey_r23_upper
    ramsey_number_upper(Nat.2, Nat.3, Nat.3)
    ramsey_r23_lower_bound
    exists(color: (Nat, Nat) -> Bool) {
        is_edge_2_coloring(complete_graph[Nat], color) and
        not has_red_clique(color, range_set(Nat.2), Nat.2) and
        not has_blue_clique(color, range_set(Nat.2), Nat.3)
    }
    ramsey_number_upper(Nat.2, Nat.3, Nat.3) and
    exists(color: (Nat, Nat) -> Bool) {
        is_edge_2_coloring(complete_graph[Nat], color) and
        not has_red_clique(color, range_set(Nat.2), Nat.2) and
        not has_blue_clique(color, range_set(Nat.2), Nat.3)
    }
}

// ============================================================================
// Section B: R(3, 3) = 6.
//
// The upper bound R(3, 3) <= 6 is verified in graph/ramsey.ac (the triangle
// form: every 2-coloring of K6 has a monochromatic triangle) and restated in
// the clique form in graph/ramsey_numbers.ac.  Both are restated here.  The
// lower bound R(3, 3) > 5 (the C5-coloring witness, no monochromatic triangle
// on five vertices) is recorded as a comment in graph/ramsey_lower_bound.ac;
// its case analysis is not yet verified, so the equality R(3, 3) = 6 is
// recorded as a comment at the end of this file.
// ============================================================================

/// R(3, 3) <= 6, the library's triangle form, restated: every 2-coloring of
/// the complete graph on the six vertices has a monochromatic triangle.
theorem ramsey_r33_upper_restated(color: (Nat, Nat) -> Bool) {
    is_edge_2_coloring(complete_graph[Nat], color)
    implies has_mono_triangle(color, six_vertices)
} by {
    if is_edge_2_coloring(complete_graph[Nat], color) {
        ramsey_r33_upper(color)
        has_mono_triangle(color, six_vertices)
    }
}

/// R(3, 3) <= 6 in the clique form `ramsey_number_upper(3, 3, 6)`: every
/// 2-coloring of the complete graph on six vertices has a red clique of size
/// three or a blue clique of size three.
theorem ramsey_r33_upper_clique {
    ramsey_number_upper(Nat.3, Nat.3, Nat.6)
} by {
    ramsey_r33_upper_number
    ramsey_number_upper(Nat.3, Nat.3, Nat.6)
}

// ============================================================================
// Section C: R(3, 4) <= 9.
//
// graph/ramsey34_upper.ac verifies, by the pigeonhole on the eight edges from
// vertex 0, that every 2-coloring of K9 has a monochromatic triangle or a
// monochromatic K4.  The pigeonhole step and the theorem are restated here.
// The asymmetric form "a red triangle or a blue K4" (which is the content of
// `ramsey_number_upper(3, 4, 9)` in the framework of ramsey_numbers.ac) needs
// the degree/parity argument whose R(3, 3) <= 6 step must be transported to an
// arbitrary six-vertex subset; that step is not yet verified, and the statement
// is recorded as a comment at the end of this file.
// ============================================================================

/// The pigeonhole step of the R(3, 4) <= 9 proof, restated: among the eight
/// edges from vertex 0 of K9, four share a color.
theorem ramsey_r34_pigeonhole_restated(color: (Nat, Nat) -> Bool) {
    four_edges_from_zero_share_color(color)
} by {
    four_edges_from_zero_share_color_pigeonhole(color)
    four_edges_from_zero_share_color(color)
}

/// R(3, 4) <= 9, restated: every red/blue 2-coloring of the complete graph on
/// the nine vertices has a monochromatic triangle or a monochromatic K4.
theorem ramsey_r34_upper_restated(color: (Nat, Nat) -> Bool) {
    is_edge_2_coloring(complete_graph[Nat], color) implies
    (has_mono_triangle(color, nine_vertices) or has_mono_k4(color, nine_vertices))
} by {
    if is_edge_2_coloring(complete_graph[Nat], color) {
        ramsey_r34_upper(color)
        has_mono_triangle(color, nine_vertices) or has_mono_k4(color, nine_vertices)
    }
}

// ============================================================================
// Section D: R(3, 5) <= 14.
//
// The recurrence R(k, l) <= R(k - 1, l) + R(k, l - 1) at (3, 5) with a = 5 and
// b = 9 reduces R(3, 5) to R(2, 5) + R(3, 4) = 5 + 9 = 14.  The arithmetic and
// the reduction through the recurrence are verified in graph/ramsey_numbers.ac
// and restated here; the sub-bounds R(2, 5) <= 5 and R(3, 4) <= 9 and the
// recurrence itself are discussed below (R(2, 5) <= 5 is an instance of the
// general R(2, k) <= k proved in Section A).
// ============================================================================

/// R(2, 5) <= 5: the instance k = 5 of the general upper bound.
theorem ramsey_r25_upper {
    ramsey_number_upper(Nat.2, Nat.5, Nat.5)
} by {
    ramsey_r2k_upper(Nat.5)
    ramsey_number_upper(Nat.2, Nat.5, Nat.5)
}

/// R(3, 5) <= 14: the arithmetic content 5 + 9 = 14, restated.
theorem ramsey_r35_bound_arith_restated {
    Nat.14 = Nat.5 + Nat.9
} by {
    ramsey_r35_bound_arith
    Nat.14 = Nat.5 + Nat.9
}

/// R(3, 5) <= R(2, 5) + R(3, 4) = 5 + 9 = 14 through the recurrence, restated:
/// granted the two sub-bounds and the recurrence, R(3, 5) <= 14.
theorem ramsey_r35_from_recurrence_restated {
    ramsey_number_upper(Nat.2, Nat.5, Nat.5) and
    ramsey_number_upper(Nat.3, Nat.4, Nat.9) and
    ramsey_recurrence(Nat.3, Nat.5, Nat.5, Nat.9)
    implies ramsey_number_upper(Nat.3, Nat.5, Nat.14)
} by {
    if ramsey_number_upper(Nat.2, Nat.5, Nat.5) and
        ramsey_number_upper(Nat.3, Nat.4, Nat.9) and
        ramsey_recurrence(Nat.3, Nat.5, Nat.5, Nat.9) {
        ramsey_r35_from_recurrence
        ramsey_number_upper(Nat.3, Nat.5, Nat.14)
    }
}

// ============================================================================
// Section E: the general bound R(k, l) <= C(k + l - 2, k - 1).
//
// The binomial bound follows from the recurrence R(k, l) <= R(k - 1, l) +
// R(k, l - 1) by induction on k + l with the Pascal identity; its full proof
// needs the recurrence itself, which is not yet verified (see the comments at
// the end of this file).  The small instances R(2, 3) <= C(3, 1) = 3 and
// R(3, 3) <= C(4, 2) = 6 are verified below.
// ============================================================================

/// The general upper bound R(k, l) <= C(k + l - 2, k - 1), in predicate form.
define ramsey_binom_bound(k: Nat, l: Nat) -> Bool {
    ramsey_number_upper(k, l, (k + l - Nat.2).binom(k - Nat.1))
}

/// R(2, 3) <= C(3, 1) = 3: the binomial bound at (2, 3), where
/// C(2 + 3 - 2, 2 - 1) = C(3, 1) = 3.
theorem ramsey_binom_bound_small23 {
    ramsey_binom_bound(Nat.2, Nat.3)
} by {
    ramsey_binom_bound(Nat.2, Nat.3) = ramsey_number_upper(Nat.2, Nat.3, (Nat.2 + Nat.3 - Nat.2).binom(Nat.2 - Nat.1))
    Nat.2 + Nat.3 = Nat.5
    Nat.5 - Nat.2 = Nat.3
    Nat.2 - Nat.1 = Nat.1
    (Nat.2 + Nat.3 - Nat.2).binom(Nat.2 - Nat.1) = Nat.3.binom(Nat.1)
    choose_one(Nat.3)
    Nat.3.binom(Nat.1) = Nat.3
    Nat.3 = Nat.3.binom(Nat.1)
    ramsey_number_upper(Nat.2, Nat.3, Nat.3) = ramsey_number_upper(Nat.2, Nat.3, Nat.3.binom(Nat.1))
    ramsey_r23_upper
    ramsey_number_upper(Nat.2, Nat.3, Nat.3)
    ramsey_number_upper(Nat.2, Nat.3, Nat.3.binom(Nat.1))
    ramsey_binom_bound(Nat.2, Nat.3)
}

/// R(3, 3) <= C(4, 2) = 6: the binomial bound at (3, 3), where
/// C(3 + 3 - 2, 3 - 1) = C(4, 2) = 6.
theorem ramsey_binom_bound_small33 {
    ramsey_binom_bound(Nat.3, Nat.3)
} by {
    ramsey_binom_bound(Nat.3, Nat.3) = ramsey_number_upper(Nat.3, Nat.3, (Nat.3 + Nat.3 - Nat.2).binom(Nat.3 - Nat.1))
    Nat.3 + Nat.3 = Nat.6
    Nat.6 - Nat.2 = Nat.4
    Nat.3 - Nat.1 = Nat.2
    (Nat.3 + Nat.3 - Nat.2).binom(Nat.3 - Nat.1) = Nat.4.binom(Nat.2)
    Nat.2 + Nat.2 = Nat.4
    exists_intro(function(c: Nat) { Nat.2 + c = Nat.4 }, Nat.2)
    exists(c: Nat) { Nat.2 + c = Nat.4 }
    Nat.2 <= Nat.4
    binom_two_double(Nat.4)
    Nat.2 * Nat.4.binom(Nat.2) = Nat.4 * (Nat.4 - Nat.1)
    Nat.4 - Nat.1 = Nat.3
    Nat.4 * Nat.3 = Nat.12
    Nat.2 * Nat.4.binom(Nat.2) = Nat.12
    Nat.2 * Nat.6 = Nat.12
    Nat.2 * Nat.4.binom(Nat.2) = Nat.2 * Nat.6
    Nat.2 != Nat.0
    mul_cancel_left(Nat.2, Nat.4.binom(Nat.2), Nat.6)
    Nat.4.binom(Nat.2) = Nat.6
    Nat.6 = Nat.4.binom(Nat.2)
    ramsey_number_upper(Nat.3, Nat.3, Nat.6) = ramsey_number_upper(Nat.3, Nat.3, Nat.4.binom(Nat.2))
    ramsey_r33_upper_clique
    ramsey_number_upper(Nat.3, Nat.3, Nat.6)
    ramsey_number_upper(Nat.3, Nat.3, Nat.4.binom(Nat.2))
    ramsey_binom_bound(Nat.3, Nat.3)
}

// ============================================================================
// Statements whose full proofs are not yet verified.
//
// Per the project convention, theorems that are not yet proved are recorded
// here as comments rather than as axioms.  Each comment states the theorem,
// the classical proof, and which ingredient is missing.
// ============================================================================

// ----------------------------------------------------------------------------
// R(3, 3) = 6, the equality.
//
//     theorem ramsey_r33_exact {
//         ramsey_number_upper(Nat.3, Nat.3, Nat.6) and
//         exists(color: (Nat, Nat) -> Bool) {
//             is_edge_2_coloring(complete_graph[Nat], color) and
//             not has_red_clique(color, range_set(Nat.5), Nat.3) and
//             not has_blue_clique(color, range_set(Nat.5), Nat.3)
//         }
//     }
//
// The upper bound R(3, 3) <= 6 is verified above (`ramsey_r33_upper_restated`
// in the triangle form, `ramsey_r33_upper_clique` in the clique form).  The
// lower bound R(3, 3) > 5 is the classical C5-coloring witness: the red graph
// is the 5-cycle on {0, ..., 4} and the blue graph its complement, neither of
// which contains a triangle.  The coloring `c5_color` and the verification
// that it is an edge 2-coloring live in graph/ramsey_c5.ac and
// graph/ramsey_lower_bound.ac, but the case analysis that no three vertices
// form a monochromatic triangle is not yet verified there (the proof search
// does not close the per-vertex implications reliably), so the equality is
// recorded only as a comment.
// ----------------------------------------------------------------------------

// ----------------------------------------------------------------------------
// R(3, 4) <= 9, asymmetric form.
//
//     theorem ramsey_r34_asym {
//         ramsey_number_upper(Nat.3, Nat.4, Nat.9)
//     }
//
// The library's `ramsey_r34_upper` (restated above) verifies the symmetric
// form "a monochromatic triangle or a monochromatic K4" by the 4-of-8
// pigeonhole on the edges from vertex 0.  The asymmetric form "a red triangle
// or a blue K4" is the content of `ramsey_number_upper(3, 4, 9)` in the
// framework of ramsey_numbers.ac.  The classical proof goes by contradiction
// through degree bounds: in a coloring with no red triangle and no blue K4,
// every vertex has red degree at most 3 (four red neighbours would span a
// blue K4 or close a red triangle with the vertex) and blue degree at most 5
// (six blue neighbours contain a monochromatic triangle by R(3, 3) <= 6,
// which closes with the vertex); hence every vertex has red degree exactly 3,
// so the sum of the red degrees is 9 * 3 = 27, but the handshake lemma forces
// that sum to be twice the number of red edges, hence even: contradiction.
// The missing ingredient is applying R(3, 3) <= 6 to an arbitrary six-vertex
// subset — the library states it over the fixed vertex set `six_vertices` —
// which needs transporting the statement along an order-preserving bijection;
// that step is not yet verified (see the recurrence discussion in
// graph/ramsey_numbers.ac).
// ----------------------------------------------------------------------------

// ----------------------------------------------------------------------------
// R(3, 5) <= 14.
//
//     theorem ramsey_r35_upper {
//         ramsey_number_upper(Nat.3, Nat.5, Nat.14)
//     }
//
// The classical proof is the recurrence at (k, l) = (3, 5) with a = 5, b = 9,
// using R(2, 5) <= 5 and R(3, 4) <= 9: R(3, 5) <= 5 + 9 = 14.  The arithmetic
// 5 + 9 = 14 is `ramsey_r35_bound_arith_restated` and the reduction through
// the recurrence is `ramsey_r35_from_recurrence_restated` (both verified
// above, granted the two sub-bounds), and R(2, 5) <= 5 is now verified as the
// instance k = 5 of Section A (`ramsey_r25_upper`).  The remaining ingredients
// are the recurrence itself — its proof needs the neighbourhood-transport
// step described in graph/ramsey_numbers.ac — and the asymmetric
// R(3, 4) <= 9 sub-bound (see above).
// ----------------------------------------------------------------------------

// ----------------------------------------------------------------------------
// R(2, k) = k, general lower bound.
//
//     theorem ramsey_r2k_lower(k: Nat) {
//         exists(color: (Nat, Nat) -> Bool) {
//             is_edge_2_coloring(complete_graph[Nat], color) and
//             not has_red_clique(color, range_set(k - Nat.1), Nat.2) and
//             not has_blue_clique(color, range_set(k - Nat.1), k)
//         }
//     }
//
// The upper bound R(2, k) <= k is verified for every k (`ramsey_r2k_upper`),
// and the case k = 3 of the lower bound is verified (`ramsey_r23_lower_bound`
// and `ramsey_r23_exact`).  The general lower bound uses the constantly blue
// coloring on k - 1 vertices: it has no red edge (hence no red K2), and the
// missing piece is the pigeonhole fact that a unique list of length k cannot
// have all its members in a (k - 1)-element set; that general counting step
// is not yet verified, so only the k = 3 case is stated as a theorem above.
// ----------------------------------------------------------------------------

// ----------------------------------------------------------------------------
// The general binomial bound R(k, l) <= C(k + l - 2, k - 1).
//
//     theorem ramsey_general_bound(k: Nat, l: Nat) {
//         ramsey_binom_bound(k, l)
//     }
//
// The classical proof is by induction on k + l with the base cases
// R(1, l) = 1 and R(k, 1) = 1 (a clique of size one is a single vertex), and
// the step
//
//     R(k, l) <= R(k - 1, l) + R(k, l - 1)
//             <= C(k + l - 3, k - 2) + C(k + l - 3, k - 1)
//              = C(k + l - 2, k - 1),
//
// where the last equality is the Pascal identity.  Both ingredients — the
// recurrence itself (whose proof is not yet verified, see above) and the
// induction over k + l — are missing, so the general bound is recorded as a
// comment.  The two small instances R(2, 3) <= C(3, 1) = 3 and
// R(3, 3) <= C(4, 2) = 6 are verified above (`ramsey_binom_bound_small23`,
// `ramsey_binom_bound_small33`).
// ----------------------------------------------------------------------------
