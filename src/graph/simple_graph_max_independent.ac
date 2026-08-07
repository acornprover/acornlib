from nat import Nat, lte_and_lt, lt_and_lte, lte_antisymm, lt_or_lte
from data.basic.logic import false_implies
from finite_set import FiniteSet, finite_set_empty_subset, finite_set_subset_contains
from data.finite.finite_set_card import fs_card, fs_card_empty
from data.finite.finite_set_card_ops import fs_card_insert_of_not_contains
from data.finite.finite_set_membership import fs_insert_contains_cases
from data.finite.finite_set_card_compare import fs_card_mono
from data.finite.finite_set_subset_intro import fs_subset_eq_intro
from graph.simple_graph import SimpleGraph
from graph.simple_graph_independent import is_independent_in, is_independent_in_apply,
    is_independent_in_intro
from graph.simple_graph_independent_domination import is_independent_subset,
    is_independent_subset_apply, is_independent_subset_intro, is_maximal_independent_in,
    is_maximal_independent_in_intro, is_independent_dominating_set,
    maximal_independent_is_independent_dominating
from data.nat.nat_bounded_max import is_max, has_max, is_max_apply, is_max_is_upper_bound,
    is_upper_bound_of, is_upper_bound_of_intro

numerals Nat

/// The empty finite set is independent in any graph.
///
/// Vacuously, but the verifier will not accept a proof that assumes the false antecedent — it
/// reports inconsistent assumptions. Rewriting the antecedent to `false` and citing
/// `false_implies` establishes the same thing without ever assuming it. This is the idiom for
/// a vacuously true statement about the empty finite set, of which `src/` previously had none.
theorem empty_is_independent[V](g: SimpleGraph[V]) {
    is_independent_in(g, FiniteSet.empty[V])
} by {
    forall(x: V, y: V) {
        not FiniteSet.empty[V].contains(x)
        (FiniteSet.empty[V].contains(x) and FiniteSet.empty[V].contains(y)) = false
        false_implies(not g.adj(x, y))
        (false implies not g.adj(x, y)) = true
        (((FiniteSet.empty[V].contains(x) and FiniteSet.empty[V].contains(y))
            implies not g.adj(x, y)) = true)
        ((FiniteSet.empty[V].contains(x) and FiniteSet.empty[V].contains(y))
            implies not g.adj(x, y))
    }
    is_independent_in_intro(g, FiniteSet.empty[V])
    is_independent_in(g, FiniteSet.empty[V])
}

/// The empty set is an independent subset of any vertex set.
theorem empty_is_independent_subset[V](g: SimpleGraph[V], s: FiniteSet[V]) {
    is_independent_subset(g, s, FiniteSet.empty[V])
} by {
    finite_set_empty_subset(s)
    FiniteSet.empty[V].subset_eq(s)
    empty_is_independent(g)
    is_independent_in(g, FiniteSet.empty[V])
    is_independent_subset_intro(g, s, FiniteSet.empty[V])
    is_independent_subset(g, s, FiniteSet.empty[V])
}

/// True of the sizes achieved by independent subsets of `s`.
///
/// Packaged as a predicate on `Nat` so the maximum construction applies to it.
define independent_size_pred[V](g: SimpleGraph[V], s: FiniteSet[V]) -> (Nat -> Bool) {
    function(n: Nat) {
        exists(t: FiniteSet[V]) {
            is_independent_subset(g, s, t) and fs_card(t) = n
        }
    }
}

/// An independent subset achieves its own size.
theorem independent_size_pred_intro[V](
    g: SimpleGraph[V], s: FiniteSet[V], t: FiniteSet[V]
) {
    is_independent_subset(g, s, t) implies independent_size_pred(g, s)(fs_card(t))
} by {
    if is_independent_subset(g, s, t) {
        independent_size_pred(g, s)(fs_card(t)) = exists(r: FiniteSet[V]) {
            is_independent_subset(g, s, r) and fs_card(r) = fs_card(t)
        }
        exists(r: FiniteSet[V]) {
            is_independent_subset(g, s, r) and fs_card(r) = fs_card(t)
        }
        independent_size_pred(g, s)(fs_card(t))
    }
}

/// Zero is an achieved size, witnessed by the empty set.
theorem independent_size_pred_zero[V](g: SimpleGraph[V], s: FiniteSet[V]) {
    independent_size_pred(g, s)(Nat.0)
} by {
    empty_is_independent_subset(g, s)
    is_independent_subset(g, s, FiniteSet.empty[V])
    independent_size_pred_intro(g, s, FiniteSet.empty[V])
    independent_size_pred(g, s)(fs_card(FiniteSet.empty[V]))
    fs_card_empty[V]
    fs_card(FiniteSet.empty[V]) = Nat.0
    independent_size_pred(g, s)(Nat.0)
}

/// The ambient size bounds every independent size.
///
/// Independent subsets are subsets, and counting is monotone, so no achieved size exceeds the
/// size of the ambient set.
theorem independent_size_pred_bounded[V](g: SimpleGraph[V], s: FiniteSet[V]) {
    is_upper_bound_of(independent_size_pred(g, s), fs_card(s))
} by {
    forall(n: Nat) {
        if independent_size_pred(g, s)(n) {
            independent_size_pred(g, s)(n) = exists(t: FiniteSet[V]) {
                is_independent_subset(g, s, t) and fs_card(t) = n
            }
            exists(t: FiniteSet[V]) {
                is_independent_subset(g, s, t) and fs_card(t) = n
            }
            let (t: FiniteSet[V]) satisfy {
                is_independent_subset(g, s, t) and fs_card(t) = n
            }
            is_independent_subset_apply(g, s, t)
            t.subset_eq(s)
            fs_card_mono(t, s)
            fs_card(t) <= fs_card(s)
            n <= fs_card(s)
        }
        (independent_size_pred(g, s)(n) implies n <= fs_card(s))
    }
    is_upper_bound_of_intro(independent_size_pred(g, s), fs_card(s))
    is_upper_bound_of(independent_size_pred(g, s), fs_card(s))
}

/// The largest size of an independent subset of `s`.
///
/// The independence number. Well defined because the empty set is independent and every
/// independent subset is bounded by the ambient set.
let independence_number[V](g: SimpleGraph[V], s: FiniteSet[V]) -> result: Nat satisfy {
    is_max(independent_size_pred(g, s), result)
} by {
    independent_size_pred_zero(g, s)
    independent_size_pred_bounded(g, s)
    has_max(independent_size_pred(g, s), Nat.0, fs_card(s))
}

/// Some independent subset attains the independence number.
theorem independence_number_attained[V](g: SimpleGraph[V], s: FiniteSet[V]) {
    independent_size_pred(g, s)(independence_number(g, s))
} by {
    is_max(independent_size_pred(g, s), independence_number(g, s))
    is_max_apply(independent_size_pred(g, s), independence_number(g, s))
}

/// No independent subset is larger than the independence number.
theorem independence_number_is_greatest[V](
    g: SimpleGraph[V], s: FiniteSet[V], t: FiniteSet[V]
) {
    is_independent_subset(g, s, t) implies fs_card(t) <= independence_number(g, s)
} by {
    if is_independent_subset(g, s, t) {
        independent_size_pred_intro(g, s, t)
        independent_size_pred(g, s)(fs_card(t))
        is_max(independent_size_pred(g, s), independence_number(g, s))
        is_max_is_upper_bound(independent_size_pred(g, s), independence_number(g, s),
            fs_card(t))
        fs_card(t) <= independence_number(g, s)
    }
}

/// An independent subset of the largest size exists.
theorem maximum_independent_set_exists[V](g: SimpleGraph[V], s: FiniteSet[V]) {
    exists(t: FiniteSet[V]) {
        is_independent_subset(g, s, t) and fs_card(t) = independence_number(g, s)
    }
} by {
    independence_number_attained(g, s)
    independent_size_pred(g, s)(independence_number(g, s))
    independent_size_pred(g, s)(independence_number(g, s)) = exists(t: FiniteSet[V]) {
        is_independent_subset(g, s, t) and fs_card(t) = independence_number(g, s)
    }
    exists(t: FiniteSet[V]) {
        is_independent_subset(g, s, t) and fs_card(t) = independence_number(g, s)
    }
}

/// The independence number is at most the number of vertices.
theorem independence_number_le_card[V](g: SimpleGraph[V], s: FiniteSet[V]) {
    independence_number(g, s) <= fs_card(s)
} by {
    maximum_independent_set_exists(g, s)
    let (t: FiniteSet[V]) satisfy {
        is_independent_subset(g, s, t) and fs_card(t) = independence_number(g, s)
    }
    is_independent_subset_apply(g, s, t)
    t.subset_eq(s)
    fs_card_mono(t, s)
    fs_card(t) <= fs_card(s)
    independence_number(g, s) <= fs_card(s)
}

/// An independent subset of the largest size admits no new vertex.
///
/// Adding a vertex outside it would give an independent subset one larger, which the
/// independence number forbids. So a maximum independent set is a maximal one.
theorem maximum_independent_is_maximal[V](
    g: SimpleGraph[V], s: FiniteSet[V], t: FiniteSet[V]
) {
    is_independent_subset(g, s, t) and fs_card(t) = independence_number(g, s)
        implies is_maximal_independent_in(g, s, t)
} by {
    if is_independent_subset(g, s, t) and fs_card(t) = independence_number(g, s) {
        is_independent_subset_apply(g, s, t)
        t.subset_eq(s)
        forall(v: V) {
            if s.contains(v) and not t.contains(v) {
                if is_independent_in(g, t.insert(v)) {
                    forall(x: V) {
                        if t.insert(v).contains(x) {
                            fs_insert_contains_cases(t, v, x)
                            if x = v {
                                s.contains(x)
                            }
                            if x != v {
                                t.contains(x)
                                finite_set_subset_contains(t, s, x)
                                s.contains(x)
                            }
                            s.contains(x)
                        }
                        (t.insert(v).contains(x) implies s.contains(x))
                    }
                    fs_subset_eq_intro(t.insert(v), s)
                    t.insert(v).subset_eq(s)
                    is_independent_subset_intro(g, s, t.insert(v))
                    is_independent_subset(g, s, t.insert(v))
                    independence_number_is_greatest(g, s, t.insert(v))
                    fs_card(t.insert(v)) <= independence_number(g, s)
                    fs_card_insert_of_not_contains(t, v)
                    fs_card(t.insert(v)) = fs_card(t) + Nat.1
                    fs_card(t) + Nat.1 <= fs_card(t)
                    fs_card(t) < fs_card(t) + Nat.1
                    lte_and_lt(fs_card(t) + Nat.1, fs_card(t), fs_card(t) + Nat.1)
                    fs_card(t) + Nat.1 < fs_card(t) + Nat.1
                    false
                }
                not is_independent_in(g, t.insert(v))
            }
            (s.contains(v) and not t.contains(v)
                implies not is_independent_in(g, t.insert(v)))
        }
        is_maximal_independent_in_intro(g, s, t)
        is_maximal_independent_in(g, s, t)
    }
}

/// Every vertex set has a maximal independent subset.
///
/// The maximum one is maximal, so the greedy construction that would otherwise be needed is
/// avoided entirely. This is the existence result the independent domination number rests on.
theorem maximal_independent_set_exists[V](g: SimpleGraph[V], s: FiniteSet[V]) {
    exists(t: FiniteSet[V]) { is_maximal_independent_in(g, s, t) }
} by {
    maximum_independent_set_exists(g, s)
    let (t: FiniteSet[V]) satisfy {
        is_independent_subset(g, s, t) and fs_card(t) = independence_number(g, s)
    }
    maximum_independent_is_maximal(g, s, t)
    is_maximal_independent_in(g, s, t)
    exists(r: FiniteSet[V]) { is_maximal_independent_in(g, s, r) }
}

/// Every vertex set has an independent dominating set.
theorem independent_dominating_set_exists[V](g: SimpleGraph[V], s: FiniteSet[V]) {
    exists(t: FiniteSet[V]) { is_independent_dominating_set(g, s, t) }
} by {
    maximal_independent_set_exists(g, s)
    let (t: FiniteSet[V]) satisfy {
        is_maximal_independent_in(g, s, t)
    }
    maximal_independent_is_independent_dominating(g, s, t)
    is_independent_dominating_set(g, s, t)
    exists(r: FiniteSet[V]) { is_independent_dominating_set(g, s, r) }
}
