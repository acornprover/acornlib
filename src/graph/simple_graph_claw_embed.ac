from nat import Nat, lt_and_lte
from data.fin.fin import Fin, value_lt_bound, ext
from data.basic.functions import is_injective_fn
from finite_set import FiniteSet
from graph.simple_graph import SimpleGraph, simple_graph_adj_ne, simple_graph_adj_symmetric,
    is_graph_embedding
from graph.simple_graph_degree import neighborhood, neighborhood_contains_eq
from graph.simple_graph_claw import has_independent_neighbor_triple,
    has_independent_neighbor_triple_witness, is_claw_free, is_claw_free_intro
from graph.simple_graph_claw_graph import claw4_graph, claw4_graph_adj_iff
from graph.simple_graph_forbidden import contains_induced, contains_induced_intro,
    is_graph_embedding_of_adj_eq, is_free_of

numerals Nat

/// A natural below four is one of the four labels.
///
/// Comparisons between numerals are not free, so the ladder is walked once here rather than at
/// each of the four branches below.
theorem lt_four_cases(k: Nat) {
    k < Nat.4 implies (k = Nat.0 or k = Nat.1 or k = Nat.2 or k = Nat.3)
} by {
    if k < Nat.4 {
        (Nat.3.suc = Nat.4)
        k < Nat.3.suc
        k <= Nat.3
        if k != Nat.3 {
            k < Nat.3
            (Nat.2.suc = Nat.3)
            k < Nat.2.suc
            k <= Nat.2
            if k != Nat.2 {
                k < Nat.2
                (Nat.1.suc = Nat.2)
                k < Nat.1.suc
                k <= Nat.1
                if k != Nat.1 {
                    k < Nat.1
                    (Nat.0.suc = Nat.1)
                    k < Nat.0.suc
                    k <= Nat.0
                    k = Nat.0
                }
                (k = Nat.0 or k = Nat.1 or k = Nat.2 or k = Nat.3)
            }
            (k = Nat.0 or k = Nat.1 or k = Nat.2 or k = Nat.3)
        }
        (k = Nat.0 or k = Nat.1 or k = Nat.2 or k = Nat.3)
    }
}

/// The label of a claw vertex is one of the four.
theorem claw4_value_cases(x: Fin[Nat.4]) {
    x.value = Nat.0 or x.value = Nat.1 or x.value = Nat.2 or x.value = Nat.3
} by {
    value_lt_bound(Nat.4, x)
    x.value < Nat.4
    lt_four_cases(x.value)
    (x.value = Nat.0 or x.value = Nat.1 or x.value = Nat.2 or x.value = Nat.3)
}

/// Claw vertices with the same label are equal.
theorem claw4_eq_of_value_eq(x: Fin[Nat.4], y: Fin[Nat.4]) {
    x.value = y.value implies x = y
} by {
    if x.value = y.value {
        ext(Nat.4, x, y)
        x = y
    }
}

/// The map carrying the claw onto a center and three independent neighbors.
///
/// The center goes to `a` and the three leaves to `b`, `c`, and `d` in order. Written as nested
/// conditionals on the label, which is the only place the four cases appear; every statement
/// below reads the map through the four equations rather than through the conditional.
define claw_map[V](a: V, b: V, c: V, d: V) -> (Fin[Nat.4] -> V) {
    function(x: Fin[Nat.4]) {
        if x.value = Nat.0 {
            a
        } else {
            if x.value = Nat.1 {
                b
            } else {
                if x.value = Nat.2 {
                    c
                } else {
                    d
                }
            }
        }
    }
}

/// The map at the center.
theorem claw_map_zero[V](a: V, b: V, c: V, d: V, x: Fin[Nat.4]) {
    x.value = Nat.0 implies claw_map(a, b, c, d)(x) = a
}

/// The map at the first leaf.
theorem claw_map_one[V](a: V, b: V, c: V, d: V, x: Fin[Nat.4]) {
    x.value = Nat.1 implies claw_map(a, b, c, d)(x) = b
} by {
    if x.value = Nat.1 {
        (Nat.0.suc = Nat.1)
        Nat.0 < Nat.1
        x.value != Nat.0
        claw_map(a, b, c, d)(x) = b
    }
}

/// The map at the second leaf.
theorem claw_map_two[V](a: V, b: V, c: V, d: V, x: Fin[Nat.4]) {
    x.value = Nat.2 implies claw_map(a, b, c, d)(x) = c
} by {
    if x.value = Nat.2 {
        (Nat.1.suc = Nat.2)
        Nat.1 < Nat.2
        (Nat.0.suc = Nat.1)
        Nat.0 < Nat.1
        lt_and_lte(Nat.0, Nat.1, Nat.2)
        Nat.0 < Nat.2
        x.value != Nat.0
        x.value != Nat.1
        claw_map(a, b, c, d)(x) = c
    }
}

/// The map at the third leaf.
theorem claw_map_three[V](a: V, b: V, c: V, d: V, x: Fin[Nat.4]) {
    x.value = Nat.3 implies claw_map(a, b, c, d)(x) = d
} by {
    if x.value = Nat.3 {
        (Nat.2.suc = Nat.3)
        Nat.2 < Nat.3
        (Nat.1.suc = Nat.2)
        Nat.1 < Nat.2
        (Nat.0.suc = Nat.1)
        Nat.0 < Nat.1
        lt_and_lte(Nat.1, Nat.2, Nat.3)
        Nat.1 < Nat.3
        lt_and_lte(Nat.0, Nat.1, Nat.3)
        Nat.0 < Nat.3
        x.value != Nat.0
        x.value != Nat.1
        x.value != Nat.2
        claw_map(a, b, c, d)(x) = d
    }
}

/// Away from the center, the map lands on one of the three leaves.
theorem claw_map_leaf[V](a: V, b: V, c: V, d: V, x: Fin[Nat.4]) {
    x.value != Nat.0 implies (claw_map(a, b, c, d)(x) = b
        or claw_map(a, b, c, d)(x) = c or claw_map(a, b, c, d)(x) = d)
} by {
    if x.value != Nat.0 {
        claw4_value_cases(x)
        (x.value = Nat.0 or x.value = Nat.1 or x.value = Nat.2 or x.value = Nat.3)
        if x.value = Nat.1 {
            claw_map_one(a, b, c, d, x)
            claw_map(a, b, c, d)(x) = b
        }
        if x.value = Nat.2 {
            claw_map_two(a, b, c, d, x)
            claw_map(a, b, c, d)(x) = c
        }
        if x.value = Nat.3 {
            claw_map_three(a, b, c, d, x)
            claw_map(a, b, c, d)(x) = d
        }
        (claw_map(a, b, c, d)(x) = b
            or claw_map(a, b, c, d)(x) = c or claw_map(a, b, c, d)(x) = d)
    }
}

/// The map, label by label.
///
/// The four equations gathered into one statement, so that a step needing to know where a
/// vertex goes gets the correlation with its label rather than the image alone.
theorem claw_map_cases[V](a: V, b: V, c: V, d: V, x: Fin[Nat.4]) {
    (x.value = Nat.0 and claw_map(a, b, c, d)(x) = a)
        or (x.value = Nat.1 and claw_map(a, b, c, d)(x) = b)
        or (x.value = Nat.2 and claw_map(a, b, c, d)(x) = c)
        or (x.value = Nat.3 and claw_map(a, b, c, d)(x) = d)
} by {
    claw4_value_cases(x)
    (x.value = Nat.0 or x.value = Nat.1 or x.value = Nat.2 or x.value = Nat.3)
    if x.value = Nat.0 {
        claw_map_zero(a, b, c, d, x)
        (x.value = Nat.0 and claw_map(a, b, c, d)(x) = a)
        ((x.value = Nat.0 and claw_map(a, b, c, d)(x) = a)
            or (x.value = Nat.1 and claw_map(a, b, c, d)(x) = b)
            or (x.value = Nat.2 and claw_map(a, b, c, d)(x) = c)
            or (x.value = Nat.3 and claw_map(a, b, c, d)(x) = d))
    }
    if x.value = Nat.1 {
        claw_map_one(a, b, c, d, x)
        (x.value = Nat.1 and claw_map(a, b, c, d)(x) = b)
        ((x.value = Nat.0 and claw_map(a, b, c, d)(x) = a)
            or (x.value = Nat.1 and claw_map(a, b, c, d)(x) = b)
            or (x.value = Nat.2 and claw_map(a, b, c, d)(x) = c)
            or (x.value = Nat.3 and claw_map(a, b, c, d)(x) = d))
    }
    if x.value = Nat.2 {
        claw_map_two(a, b, c, d, x)
        (x.value = Nat.2 and claw_map(a, b, c, d)(x) = c)
        ((x.value = Nat.0 and claw_map(a, b, c, d)(x) = a)
            or (x.value = Nat.1 and claw_map(a, b, c, d)(x) = b)
            or (x.value = Nat.2 and claw_map(a, b, c, d)(x) = c)
            or (x.value = Nat.3 and claw_map(a, b, c, d)(x) = d))
    }
    if x.value = Nat.3 {
        claw_map_three(a, b, c, d, x)
        (x.value = Nat.3 and claw_map(a, b, c, d)(x) = d)
        ((x.value = Nat.0 and claw_map(a, b, c, d)(x) = a)
            or (x.value = Nat.1 and claw_map(a, b, c, d)(x) = b)
            or (x.value = Nat.2 and claw_map(a, b, c, d)(x) = c)
            or (x.value = Nat.3 and claw_map(a, b, c, d)(x) = d))
    }
    ((x.value = Nat.0 and claw_map(a, b, c, d)(x) = a)
        or (x.value = Nat.1 and claw_map(a, b, c, d)(x) = b)
        or (x.value = Nat.2 and claw_map(a, b, c, d)(x) = c)
        or (x.value = Nat.3 and claw_map(a, b, c, d)(x) = d))
}

/// True if four vertices are pairwise distinct.
///
/// Named so that the injectivity statement is not six conjuncts deep.
define four_distinct[V](a: V, b: V, c: V, d: V) -> Bool {
    a != b and a != c and a != d and b != c and b != d and c != d
}

/// The map is injective when its four values are distinct.
theorem claw_map_injective[V](a: V, b: V, c: V, d: V) {
    four_distinct(a, b, c, d) implies is_injective_fn(claw_map(a, b, c, d))
} by {
    if four_distinct(a, b, c, d) {
        (four_distinct(a, b, c, d)
            = (a != b and a != c and a != d and b != c and b != d and c != d))
        forall(x: Fin[Nat.4], y: Fin[Nat.4]) {
            if claw_map(a, b, c, d)(x) = claw_map(a, b, c, d)(y) {
                claw4_value_cases(x)
                (x.value = Nat.0 or x.value = Nat.1 or x.value = Nat.2
                    or x.value = Nat.3)
                claw4_value_cases(y)
                (y.value = Nat.0 or y.value = Nat.1 or y.value = Nat.2
                    or y.value = Nat.3)
                if x.value = Nat.0 {
                    claw_map_zero(a, b, c, d, x)
                    claw_map(a, b, c, d)(x) = a
                    if y.value = Nat.0 {
                        x.value = y.value
                    }
                    if y.value = Nat.1 {
                        claw_map_one(a, b, c, d, y)
                        claw_map(a, b, c, d)(y) = b
                        a = b
                        false
                    }
                    if y.value = Nat.2 {
                        claw_map_two(a, b, c, d, y)
                        claw_map(a, b, c, d)(y) = c
                        a = c
                        false
                    }
                    if y.value = Nat.3 {
                        claw_map_three(a, b, c, d, y)
                        claw_map(a, b, c, d)(y) = d
                        a = d
                        false
                    }
                    x.value = y.value
                }
                if x.value = Nat.1 {
                    claw_map_one(a, b, c, d, x)
                    claw_map(a, b, c, d)(x) = b
                    if y.value = Nat.0 {
                        claw_map_zero(a, b, c, d, y)
                        claw_map(a, b, c, d)(y) = a
                        b = a
                        false
                    }
                    if y.value = Nat.1 {
                        x.value = y.value
                    }
                    if y.value = Nat.2 {
                        claw_map_two(a, b, c, d, y)
                        claw_map(a, b, c, d)(y) = c
                        b = c
                        false
                    }
                    if y.value = Nat.3 {
                        claw_map_three(a, b, c, d, y)
                        claw_map(a, b, c, d)(y) = d
                        b = d
                        false
                    }
                    x.value = y.value
                }
                if x.value = Nat.2 {
                    claw_map_two(a, b, c, d, x)
                    claw_map(a, b, c, d)(x) = c
                    if y.value = Nat.0 {
                        claw_map_zero(a, b, c, d, y)
                        claw_map(a, b, c, d)(y) = a
                        c = a
                        false
                    }
                    if y.value = Nat.1 {
                        claw_map_one(a, b, c, d, y)
                        claw_map(a, b, c, d)(y) = b
                        c = b
                        false
                    }
                    if y.value = Nat.2 {
                        x.value = y.value
                    }
                    if y.value = Nat.3 {
                        claw_map_three(a, b, c, d, y)
                        claw_map(a, b, c, d)(y) = d
                        c = d
                        false
                    }
                    x.value = y.value
                }
                if x.value = Nat.3 {
                    claw_map_three(a, b, c, d, x)
                    claw_map(a, b, c, d)(x) = d
                    if y.value = Nat.0 {
                        claw_map_zero(a, b, c, d, y)
                        claw_map(a, b, c, d)(y) = a
                        d = a
                        false
                    }
                    if y.value = Nat.1 {
                        claw_map_one(a, b, c, d, y)
                        claw_map(a, b, c, d)(y) = b
                        d = b
                        false
                    }
                    if y.value = Nat.2 {
                        claw_map_two(a, b, c, d, y)
                        claw_map(a, b, c, d)(y) = c
                        d = c
                        false
                    }
                    if y.value = Nat.3 {
                        x.value = y.value
                    }
                    x.value = y.value
                }
                x.value = y.value
                claw4_eq_of_value_eq(x, y)
                x = y
            }
            (claw_map(a, b, c, d)(x) = claw_map(a, b, c, d)(y) implies x = y)
        }
        (is_injective_fn(claw_map(a, b, c, d)) = forall(x: Fin[Nat.4], y: Fin[Nat.4]) {
            claw_map(a, b, c, d)(x) = claw_map(a, b, c, d)(y) implies x = y
        })
        is_injective_fn(claw_map(a, b, c, d))
    }
}

/// The six further non-adjacencies among the three leaves.
///
/// Three from symmetry and three from irreflexivity. Supplied as a separate statement so that
/// the case analysis downstream cites nine plain facts rather than discharging a conjunction
/// of two disjunctions, which proof search does not manage.
theorem leaves_not_adj_extra[V](g: SimpleGraph[V], b: V, c: V, d: V) {
    not g.adj(b, c) and not g.adj(b, d) and not g.adj(c, d)
        implies not g.adj(c, b) and not g.adj(d, b) and not g.adj(d, c)
            and not g.adj(b, b) and not g.adj(c, c) and not g.adj(d, d)
} by {
    if not g.adj(b, c) and not g.adj(b, d) and not g.adj(c, d) {
        simple_graph_adj_symmetric(g, b, c)
        (g.adj(b, c) = g.adj(c, b))
        not g.adj(c, b)
        simple_graph_adj_symmetric(g, b, d)
        (g.adj(b, d) = g.adj(d, b))
        not g.adj(d, b)
        simple_graph_adj_symmetric(g, c, d)
        (g.adj(c, d) = g.adj(d, c))
        not g.adj(d, c)
        if g.adj(b, b) {
            simple_graph_adj_ne(g, b, b)
            b != b
            false
        }
        not g.adj(b, b)
        if g.adj(c, c) {
            simple_graph_adj_ne(g, c, c)
            c != c
            false
        }
        not g.adj(c, c)
        if g.adj(d, d) {
            simple_graph_adj_ne(g, d, d)
            d != d
            false
        }
        not g.adj(d, d)
        (not g.adj(c, b) and not g.adj(d, b) and not g.adj(d, c)
            and not g.adj(b, b) and not g.adj(c, c) and not g.adj(d, d))
    }
}

/// A center with three independent neighbors carries an induced claw.
///
/// The map sends the center of the claw to `a` and its three leaves to `b`, `c`, and `d`. It
/// reflects adjacency as well as preserving it, which is what makes the copy induced: two
/// leaves are non-adjacent in the claw and their images are non-adjacent in `g`.
theorem claw_map_is_embedding[V](g: SimpleGraph[V], a: V, b: V, c: V, d: V) {
    g.adj(a, b) and g.adj(a, c) and g.adj(a, d)
        and b != c and b != d and c != d
        and not g.adj(b, c) and not g.adj(b, d) and not g.adj(c, d)
        implies is_graph_embedding(claw4_graph, g, claw_map(a, b, c, d))
} by {
    if g.adj(a, b) and g.adj(a, c) and g.adj(a, d)
        and b != c and b != d and c != d
        and not g.adj(b, c) and not g.adj(b, d) and not g.adj(c, d) {
        simple_graph_adj_ne(g, a, b)
        a != b
        simple_graph_adj_ne(g, a, c)
        a != c
        simple_graph_adj_ne(g, a, d)
        a != d
        (four_distinct(a, b, c, d)
            = (a != b and a != c and a != d and b != c and b != d and c != d))
        four_distinct(a, b, c, d)
        claw_map_injective(a, b, c, d)
        is_injective_fn(claw_map(a, b, c, d))
        if g.adj(a, a) {
            simple_graph_adj_ne(g, a, a)
            a != a
            false
        }
        not g.adj(a, a)
        leaves_not_adj_extra(g, b, c, d)
        (not g.adj(c, b) and not g.adj(d, b) and not g.adj(d, c)
            and not g.adj(b, b) and not g.adj(c, c) and not g.adj(d, d))
        simple_graph_adj_symmetric(g, a, b)
        (g.adj(a, b) = g.adj(b, a))
        g.adj(b, a)
        simple_graph_adj_symmetric(g, a, c)
        (g.adj(a, c) = g.adj(c, a))
        g.adj(c, a)
        simple_graph_adj_symmetric(g, a, d)
        (g.adj(a, d) = g.adj(d, a))
        g.adj(d, a)
        forall(x: Fin[Nat.4], y: Fin[Nat.4]) {
            claw4_graph_adj_iff(x, y)
            (claw4_graph.adj(x, y) = ((x.value = Nat.0 and y.value != Nat.0)
                or (y.value = Nat.0 and x.value != Nat.0)))
            if x.value = Nat.0 {
                claw_map_zero(a, b, c, d, x)
                claw_map(a, b, c, d)(x) = a
                if y.value = Nat.0 {
                    claw4_eq_of_value_eq(x, y)
                    x = y
                    claw_map(a, b, c, d)(y) = a
                    not g.adj(claw_map(a, b, c, d)(x), claw_map(a, b, c, d)(y))
                    not claw4_graph.adj(x, y)
                    (g.adj(claw_map(a, b, c, d)(x), claw_map(a, b, c, d)(y))
                        = claw4_graph.adj(x, y))
                }
                if y.value != Nat.0 {
                    claw_map_leaf(a, b, c, d, y)
                    (claw_map(a, b, c, d)(y) = b or claw_map(a, b, c, d)(y) = c
                        or claw_map(a, b, c, d)(y) = d)
                    if claw_map(a, b, c, d)(y) = b {
                        g.adj(a, claw_map(a, b, c, d)(y))
                    }
                    if claw_map(a, b, c, d)(y) = c {
                        g.adj(a, claw_map(a, b, c, d)(y))
                    }
                    if claw_map(a, b, c, d)(y) = d {
                        g.adj(a, claw_map(a, b, c, d)(y))
                    }
                    g.adj(a, claw_map(a, b, c, d)(y))
                    g.adj(claw_map(a, b, c, d)(x), claw_map(a, b, c, d)(y))
                    claw4_graph.adj(x, y)
                    (g.adj(claw_map(a, b, c, d)(x), claw_map(a, b, c, d)(y))
                        = claw4_graph.adj(x, y))
                }
                (g.adj(claw_map(a, b, c, d)(x), claw_map(a, b, c, d)(y))
                    = claw4_graph.adj(x, y))
            }
            if x.value != Nat.0 {
                claw_map_leaf(a, b, c, d, x)
                (claw_map(a, b, c, d)(x) = b or claw_map(a, b, c, d)(x) = c
                    or claw_map(a, b, c, d)(x) = d)
                if y.value = Nat.0 {
                    claw_map_zero(a, b, c, d, y)
                    claw_map(a, b, c, d)(y) = a
                    if claw_map(a, b, c, d)(x) = b {
                        g.adj(claw_map(a, b, c, d)(x), a)
                    }
                    if claw_map(a, b, c, d)(x) = c {
                        g.adj(claw_map(a, b, c, d)(x), a)
                    }
                    if claw_map(a, b, c, d)(x) = d {
                        g.adj(claw_map(a, b, c, d)(x), a)
                    }
                    g.adj(claw_map(a, b, c, d)(x), a)
                    g.adj(claw_map(a, b, c, d)(x), claw_map(a, b, c, d)(y))
                    claw4_graph.adj(x, y)
                    (g.adj(claw_map(a, b, c, d)(x), claw_map(a, b, c, d)(y))
                        = claw4_graph.adj(x, y))
                }
                if y.value != Nat.0 {
                    claw_map_leaf(a, b, c, d, y)
                    (claw_map(a, b, c, d)(y) = b or claw_map(a, b, c, d)(y) = c
                        or claw_map(a, b, c, d)(y) = d)
                    if claw_map(a, b, c, d)(x) = b {
                        if claw_map(a, b, c, d)(y) = b {
                            not g.adj(b, b)
                            not g.adj(claw_map(a, b, c, d)(x), claw_map(a, b, c, d)(y))
                        }
                        if claw_map(a, b, c, d)(y) = c {
                            not g.adj(b, c)
                            not g.adj(claw_map(a, b, c, d)(x), claw_map(a, b, c, d)(y))
                        }
                        if claw_map(a, b, c, d)(y) = d {
                            not g.adj(b, d)
                            not g.adj(claw_map(a, b, c, d)(x), claw_map(a, b, c, d)(y))
                        }
                        not g.adj(claw_map(a, b, c, d)(x), claw_map(a, b, c, d)(y))
                    }
                    if claw_map(a, b, c, d)(x) = c {
                        if claw_map(a, b, c, d)(y) = b {
                            not g.adj(c, b)
                            not g.adj(claw_map(a, b, c, d)(x), claw_map(a, b, c, d)(y))
                        }
                        if claw_map(a, b, c, d)(y) = c {
                            not g.adj(c, c)
                            not g.adj(claw_map(a, b, c, d)(x), claw_map(a, b, c, d)(y))
                        }
                        if claw_map(a, b, c, d)(y) = d {
                            not g.adj(c, d)
                            not g.adj(claw_map(a, b, c, d)(x), claw_map(a, b, c, d)(y))
                        }
                        not g.adj(claw_map(a, b, c, d)(x), claw_map(a, b, c, d)(y))
                    }
                    if claw_map(a, b, c, d)(x) = d {
                        if claw_map(a, b, c, d)(y) = b {
                            not g.adj(d, b)
                            not g.adj(claw_map(a, b, c, d)(x), claw_map(a, b, c, d)(y))
                        }
                        if claw_map(a, b, c, d)(y) = c {
                            not g.adj(d, c)
                            not g.adj(claw_map(a, b, c, d)(x), claw_map(a, b, c, d)(y))
                        }
                        if claw_map(a, b, c, d)(y) = d {
                            not g.adj(d, d)
                            not g.adj(claw_map(a, b, c, d)(x), claw_map(a, b, c, d)(y))
                        }
                        not g.adj(claw_map(a, b, c, d)(x), claw_map(a, b, c, d)(y))
                    }
                    not g.adj(claw_map(a, b, c, d)(x), claw_map(a, b, c, d)(y))
                    not claw4_graph.adj(x, y)
                    (g.adj(claw_map(a, b, c, d)(x), claw_map(a, b, c, d)(y))
                        = claw4_graph.adj(x, y))
                }
                (g.adj(claw_map(a, b, c, d)(x), claw_map(a, b, c, d)(y))
                    = claw4_graph.adj(x, y))
            }
            (g.adj(claw_map(a, b, c, d)(x), claw_map(a, b, c, d)(y))
                = claw4_graph.adj(x, y))
        }
        is_graph_embedding_of_adj_eq(claw4_graph, g, claw_map(a, b, c, d))
        is_graph_embedding(claw4_graph, g, claw_map(a, b, c, d))
    }
}

/// A graph with an independent neighbor triple contains an induced claw.
theorem contains_induced_claw_of_triple[V](
    g: SimpleGraph[V], s: FiniteSet[V], a: V
) {
    has_independent_neighbor_triple(g, s, a) implies contains_induced(g, claw4_graph)
} by {
    if has_independent_neighbor_triple(g, s, a) {
        has_independent_neighbor_triple_witness(g, s, a)
        exists(x: V, y: V, z: V) {
            neighborhood(g, s, a).contains(x) and neighborhood(g, s, a).contains(y)
                and neighborhood(g, s, a).contains(z)
                and x != y and x != z and y != z
                and not g.adj(x, y) and not g.adj(x, z) and not g.adj(y, z)
        }
        let (b: V, c: V, d: V) satisfy {
            neighborhood(g, s, a).contains(b) and neighborhood(g, s, a).contains(c)
                and neighborhood(g, s, a).contains(d)
                and b != c and b != d and c != d
                and not g.adj(b, c) and not g.adj(b, d) and not g.adj(c, d)
        }
        neighborhood_contains_eq(g, s, a, b)
        (neighborhood(g, s, a).contains(b) = (s.contains(b) and g.adj(a, b)))
        g.adj(a, b)
        neighborhood_contains_eq(g, s, a, c)
        (neighborhood(g, s, a).contains(c) = (s.contains(c) and g.adj(a, c)))
        g.adj(a, c)
        neighborhood_contains_eq(g, s, a, d)
        (neighborhood(g, s, a).contains(d) = (s.contains(d) and g.adj(a, d)))
        g.adj(a, d)
        claw_map_is_embedding(g, a, b, c, d)
        is_graph_embedding(claw4_graph, g, claw_map(a, b, c, d))
        contains_induced_intro(g, claw4_graph, claw_map(a, b, c, d))
        contains_induced(g, claw4_graph)
    }
}

/// A graph free of the claw is claw-free on every vertex set.
///
/// The direction that makes the combinatorial condition usable from the graph-level one: it
/// says the four-vertex forbidden subgraph and the neighborhood condition agree.
theorem is_claw_free_of_free_of_claw[V](g: SimpleGraph[V], s: FiniteSet[V]) {
    is_free_of(g, claw4_graph) implies is_claw_free(g, s)
} by {
    if is_free_of(g, claw4_graph) {
        (is_free_of(g, claw4_graph) = not contains_induced(g, claw4_graph))
        not contains_induced(g, claw4_graph)
        forall(a: V) {
            if has_independent_neighbor_triple(g, s, a) {
                contains_induced_claw_of_triple(g, s, a)
                contains_induced(g, claw4_graph)
                false
            }
            not has_independent_neighbor_triple(g, s, a)
        }
        is_claw_free_intro(g, s)
        is_claw_free(g, s)
    }
}
