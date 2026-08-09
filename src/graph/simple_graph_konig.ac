from nat import Nat, lte_trans, lte_antisymm, lt_or_lte, lte_and_lt, lt_imp_lte_suc,
    lte_add_left, lte_add_right, lt_suc_right, not_lt_zero, is_min, has_min, is_min_apply,
    is_min_false_below, false_below, false_below_apply, lt_suc, lt_not_ref, lt_and_lte,
    lt_imp_lt_suc
from pair import Pair, pair_first, pair_second
from finite_set import FiniteSet, fs_union, fs_image, fs_insert, finite_set_empty_subset,
    finite_set_subset_contains, finite_set_subset_refl, finite_set_union_contains_eq,
    finite_set_image_contains_witness, finite_set_maps_into_image, finite_set_witness_predicate,
    finite_set_witness_intro, finite_set_witness_exists_of_witness,
    choose_from_finite_set_or_default, choose_from_finite_set_or_default_contains,
    choose_from_finite_set_or_default_property, finite_set_singleton_contains_eq,
    finite_set_singleton_subset_of_contains, finite_set_empty_contains_eq,
    finite_set_insert_cardinality_is_suc_of_not_contains
from data.finite.finite_set_membership import fs_insert_contains_eq
from data.finite.finite_set_card import fs_card, fs_card_empty, fs_card_eq_of_cardinality_is,
    fs_card_cardinality_is, fs_card_singleton
from data.finite.finite_set_card_compare import fs_card_mono
from data.finite.finite_set_subset_intro import fs_subset_eq_intro
from data.nat.nat_bounded_max import is_max, is_max_intro, is_max_apply,
    is_max_is_upper_bound, is_upper_bound_of, is_upper_bound_of_intro, max_unique
from data.basic.logic import false_implies
from data.basic.relation_basic import is_symmetric, is_irreflexive
from graph.simple_graph import SimpleGraph, simple_graph_adj_symmetric
from graph.simple_graph_edges import directed_edges, directed_edges_contains_eq,
    directed_edges_contains_pair, directed_edges_adj, directed_edges_first,
    directed_edges_second
from graph.simple_graph_matching import is_matching, is_matching_intro, is_matching_apply_edge,
    is_matching_apply_disjoint, pairs_share_vertex, matching_size_pred,
    matching_size_pred_intro, matching_size_le_half_vertices, maximum_matching_size,
    maximum_matching_size_attained, maximum_matching_exists,
    fs_card_image_eq_of_locally_injective
from graph.simple_graph_vertex_sets import is_vertex_cover, is_vertex_cover_apply,
    is_vertex_cover_intro, is_vertex_cover_self, is_vertex_cover_of_subset
from graph.simple_graph_cover_number import cover_size_pred, vertex_cover_number,
    cover_size_pred_intro, vertex_cover_number_attained
from graph.simple_graph_bipartite import is_bipartition, is_bipartition_intro,
    is_bipartite, is_bipartite_intro
from data.nat.nat_range_set import range_set, range_set_contains_eq, range_set_contains,
    range_set_lt, range_set_card

numerals Nat

// König's theorem for bipartite graphs.
//
// The library already carries the two ingredients: `is_vertex_cover` (a vertex set meeting
// every edge, in `graph/simple_graph_vertex_sets.ac`), the maximum matching size
// `maximum_matching_size(g, s)` (in `graph/simple_graph_matching.ac`), and the minimum
// vertex cover size `vertex_cover_number(g, s)` (in `graph/simple_graph_cover_number.ac`).
// In those terms König's theorem states:
//
//   is_bipartite(g, s) implies maximum_matching_size(g, s) = vertex_cover_number(g, s)
//
// This file proves the easy direction, which needs no bipartiteness: every vertex cover is
// at least as large as every matching, since a cover must contain a distinct vertex of each
// matching edge. The hard direction (a minimum vertex cover of a bipartite graph has no more
// vertices than a maximum matching) needs augmenting paths or Hall's theorem, whose
// sufficiency direction is not yet in the library; it is recorded at the end, commented out.
// The theorem is then verified on three concrete bipartite graphs (the path P3, the cycle
// C4, and the complete bipartite graph K_{2,2}).

// The easy direction: a vertex cover is at least as large as any matching.
//
// Each matching edge is assigned one of its endpoints from the cover, and the assignment is
// injective because distinct matching edges are vertex-disjoint; the injective image sits in
// the cover.

/// True of the vertices of the cover `c` that lie on the pair `p`.
define cover_vertex_pred[V](c: FiniteSet[V], p: Pair[V, V]) -> (V -> Bool) {
    function(v: V) {
        c.contains(v) and (v = p.first or v = p.second)
    }
}

/// A vertex of the cover `c` that lies on the pair `p`, or `p.first` if none exists.
define cover_vertex_of[V](c: FiniteSet[V], p: Pair[V, V]) -> V {
    choose_from_finite_set_or_default(c, cover_vertex_pred(c, p), p.first)
}

/// The chosen cover vertex of a pair, as a function on pairs.
define cover_vertex_fn[V](c: FiniteSet[V]) -> (Pair[V, V] -> V) {
    function(p: Pair[V, V]) {
        cover_vertex_of(c, p)
    }
}

/// An edge of a graph covered by `c` has a vertex of `c` on it.
theorem cover_vertex_witness_exists[V](g: SimpleGraph[V], s: FiniteSet[V], c: FiniteSet[V], p: Pair[V, V]) {
    is_vertex_cover(g, s, c) and directed_edges(g, s).contains(p) implies
        exists(v: V) {
            finite_set_witness_predicate(c, cover_vertex_pred(c, p), v)
        }
} by {
    if is_vertex_cover(g, s, c) and directed_edges(g, s).contains(p) {
        directed_edges_contains_eq(g, s, p)
        directed_edges(g, s).contains(p) = (s.contains(p.first) and s.contains(p.second) and g.adj(p.first, p.second))
        s.contains(p.first)
        s.contains(p.second)
        g.adj(p.first, p.second)
        is_vertex_cover_apply(g, s, c, p.first, p.second)
        c.contains(p.first) or c.contains(p.second)
        if c.contains(p.first) {
            cover_vertex_pred(c, p)(p.first) = (c.contains(p.first) and (p.first = p.first or p.first = p.second))
            finite_set_witness_intro(c, cover_vertex_pred(c, p), p.first)
            finite_set_witness_predicate(c, cover_vertex_pred(c, p), p.first)
            finite_set_witness_exists_of_witness(c, cover_vertex_pred(c, p), p.first)
            exists(v: V) {
                finite_set_witness_predicate(c, cover_vertex_pred(c, p), v)
            }
        }
        if not c.contains(p.first) {
            c.contains(p.second)
            cover_vertex_pred(c, p)(p.second) = (c.contains(p.second) and (p.second = p.first or p.second = p.second))
            finite_set_witness_intro(c, cover_vertex_pred(c, p), p.second)
            finite_set_witness_predicate(c, cover_vertex_pred(c, p), p.second)
            finite_set_witness_exists_of_witness(c, cover_vertex_pred(c, p), p.second)
            exists(v: V) {
                finite_set_witness_predicate(c, cover_vertex_pred(c, p), v)
            }
        }
        exists(v: V) {
            finite_set_witness_predicate(c, cover_vertex_pred(c, p), v)
        }
    }
}

/// The chosen cover vertex of an edge lies in the cover.
theorem cover_vertex_of_contains[V](g: SimpleGraph[V], s: FiniteSet[V], c: FiniteSet[V], p: Pair[V, V]) {
    is_vertex_cover(g, s, c) and directed_edges(g, s).contains(p) implies c.contains(cover_vertex_of(c, p))
} by {
    if is_vertex_cover(g, s, c) and directed_edges(g, s).contains(p) {
        cover_vertex_witness_exists(g, s, c, p)
        exists(v: V) {
            finite_set_witness_predicate(c, cover_vertex_pred(c, p), v)
        }
        choose_from_finite_set_or_default_contains(c, cover_vertex_pred(c, p), p.first)
        c.contains(cover_vertex_of(c, p))
    }
}

/// The chosen cover vertex of an edge lies on that edge.
theorem cover_vertex_of_on_pair[V](g: SimpleGraph[V], s: FiniteSet[V], c: FiniteSet[V], p: Pair[V, V]) {
    is_vertex_cover(g, s, c) and directed_edges(g, s).contains(p) implies
        (cover_vertex_of(c, p) = p.first or cover_vertex_of(c, p) = p.second)
} by {
    if is_vertex_cover(g, s, c) and directed_edges(g, s).contains(p) {
        cover_vertex_witness_exists(g, s, c, p)
        exists(v: V) {
            finite_set_witness_predicate(c, cover_vertex_pred(c, p), v)
        }
        choose_from_finite_set_or_default_property(c, cover_vertex_pred(c, p), p.first)
        cover_vertex_pred(c, p)(cover_vertex_of(c, p))
        cover_vertex_pred(c, p)(cover_vertex_of(c, p)) =
            (c.contains(cover_vertex_of(c, p)) and (cover_vertex_of(c, p) = p.first or cover_vertex_of(c, p) = p.second))
        c.contains(cover_vertex_of(c, p)) and (cover_vertex_of(c, p) = p.first or cover_vertex_of(c, p) = p.second)
        cover_vertex_of(c, p) = p.first or cover_vertex_of(c, p) = p.second
    }
}

/// Distinct matching pairs are assigned distinct cover vertices.
///
/// The chosen cover vertex of a pair lies on that pair, and two matching pairs never share
/// a vertex, so equal chosen vertices would make the two pairs share that vertex.
theorem cover_vertex_of_locally_injective[V](g: SimpleGraph[V], s: FiniteSet[V], c: FiniteSet[V],
    m: FiniteSet[Pair[V, V]], p: Pair[V, V], q: Pair[V, V]) {
    is_vertex_cover(g, s, c) and is_matching(g, s, m) and m.contains(p) and m.contains(q) and
        cover_vertex_of(c, p) = cover_vertex_of(c, q) implies p = q
} by {
    if is_vertex_cover(g, s, c) and is_matching(g, s, m) and m.contains(p) and m.contains(q) and
        cover_vertex_of(c, p) = cover_vertex_of(c, q) {
        is_matching_apply_edge(g, s, m, p)
        directed_edges(g, s).contains(p)
        is_matching_apply_edge(g, s, m, q)
        directed_edges(g, s).contains(q)
        cover_vertex_of_on_pair(g, s, c, p)
        cover_vertex_of(c, p) = p.first or cover_vertex_of(c, p) = p.second
        cover_vertex_of_on_pair(g, s, c, q)
        cover_vertex_of(c, q) = q.first or cover_vertex_of(c, q) = q.second
        if cover_vertex_of(c, p) = p.first {
            if cover_vertex_of(c, q) = q.first {
                p.first = q.first
                pairs_share_vertex(p, q) = (p.first = q.first or p.first = q.second or p.second = q.first or p.second = q.second)
                pairs_share_vertex(p, q)
                is_matching_apply_disjoint(g, s, m, p, q)
                p = q
            }
            if not (cover_vertex_of(c, q) = q.first) {
                cover_vertex_of(c, q) = q.second
                p.first = q.second
                pairs_share_vertex(p, q) = (p.first = q.first or p.first = q.second or p.second = q.first or p.second = q.second)
                pairs_share_vertex(p, q)
                is_matching_apply_disjoint(g, s, m, p, q)
                p = q
            }
            p = q
        }
        if not (cover_vertex_of(c, p) = p.first) {
            cover_vertex_of(c, p) = p.second
            if cover_vertex_of(c, q) = q.first {
                p.second = q.first
                pairs_share_vertex(p, q) = (p.first = q.first or p.first = q.second or p.second = q.first or p.second = q.second)
                pairs_share_vertex(p, q)
                is_matching_apply_disjoint(g, s, m, p, q)
                p = q
            }
            if not (cover_vertex_of(c, q) = q.first) {
                cover_vertex_of(c, q) = q.second
                p.second = q.second
                pairs_share_vertex(p, q) = (p.first = q.first or p.first = q.second or p.second = q.first or p.second = q.second)
                pairs_share_vertex(p, q)
                is_matching_apply_disjoint(g, s, m, p, q)
                p = q
            }
            p = q
        }
        p = q
    }
}

/// The easy direction of König's theorem: every vertex cover is at least as large as every
/// matching.
theorem vertex_cover_bounds_matching[V](g: SimpleGraph[V], s: FiniteSet[V], c: FiniteSet[V],
    m: FiniteSet[Pair[V, V]]) {
    is_vertex_cover(g, s, c) and is_matching(g, s, m) implies fs_card(m) <= fs_card(c)
} by {
    if is_vertex_cover(g, s, c) and is_matching(g, s, m) {
        forall(p: Pair[V, V], q: Pair[V, V]) {
            if m.contains(p) and m.contains(q) and cover_vertex_fn(c)(p) = cover_vertex_fn(c)(q) {
                cover_vertex_fn(c)(p) = cover_vertex_of(c, p)
                cover_vertex_fn(c)(q) = cover_vertex_of(c, q)
                cover_vertex_of(c, p) = cover_vertex_of(c, q)
                cover_vertex_of_locally_injective(g, s, c, m, p, q)
                p = q
            }
            m.contains(p) and m.contains(q) and cover_vertex_fn(c)(p) = cover_vertex_fn(c)(q) implies p = q
        }
        fs_card_image_eq_of_locally_injective(m, cover_vertex_fn(c))
        fs_card(fs_image(m, cover_vertex_fn(c))) = fs_card(m)
        forall(v: V) {
            if fs_image(m, cover_vertex_fn(c)).contains(v) {
                finite_set_image_contains_witness(m, cover_vertex_fn(c), v)
                let p: Pair[V, V] satisfy {
                    m.contains(p) and v = cover_vertex_fn(c)(p)
                }
                cover_vertex_fn(c)(p) = cover_vertex_of(c, p)
                v = cover_vertex_of(c, p)
                is_matching_apply_edge(g, s, m, p)
                directed_edges(g, s).contains(p)
                cover_vertex_of_contains(g, s, c, p)
                c.contains(cover_vertex_of(c, p))
                c.contains(v)
            }
            fs_image(m, cover_vertex_fn(c)).contains(v) implies c.contains(v)
        }
        fs_subset_eq_intro(fs_image(m, cover_vertex_fn(c)), c)
        fs_image(m, cover_vertex_fn(c)).subset_eq(c)
        fs_card_mono(fs_image(m, cover_vertex_fn(c)), c)
        fs_card(fs_image(m, cover_vertex_fn(c))) <= fs_card(c)
        fs_card(m) <= fs_card(c)
    }
}

/// The maximum matching size is at most the minimum vertex cover size.
///
/// The maximum matching size and the minimum vertex cover size are both achieved, so the
/// previous bound applies to a witness of each.
theorem maximum_matching_size_le_vertex_cover_number[V](g: SimpleGraph[V], s: FiniteSet[V]) {
    maximum_matching_size(g, s) <= vertex_cover_number(g, s)
} by {
    maximum_matching_size_attained(g, s)
    matching_size_pred(g, s)(maximum_matching_size(g, s))
    matching_size_pred(g, s)(maximum_matching_size(g, s)) = exists(m: FiniteSet[Pair[V, V]]) {
        is_matching(g, s, m) and fs_card(m) = maximum_matching_size(g, s)
    }
    let m: FiniteSet[Pair[V, V]] satisfy {
        is_matching(g, s, m) and fs_card(m) = maximum_matching_size(g, s)
    }
    vertex_cover_number_attained(g, s)
    cover_size_pred(g, s)(vertex_cover_number(g, s))
    cover_size_pred(g, s)(vertex_cover_number(g, s)) = exists(c: FiniteSet[V]) {
        c.subset_eq(s) and is_vertex_cover(g, s, c) and fs_card(c) = vertex_cover_number(g, s)
    }
    let c: FiniteSet[V] satisfy {
        c.subset_eq(s) and is_vertex_cover(g, s, c) and fs_card(c) = vertex_cover_number(g, s)
    }
    vertex_cover_bounds_matching(g, s, c, m)
    fs_card(m) <= fs_card(c)
    fs_card(m) = maximum_matching_size(g, s)
    maximum_matching_size(g, s) <= fs_card(c)
    fs_card(c) = vertex_cover_number(g, s)
    maximum_matching_size(g, s) <= vertex_cover_number(g, s)
}

/// The minimum vertex cover number is unique.
theorem min_unique(f: Nat -> Bool, m: Nat, m2: Nat) {
    is_min(f, m) and is_min(f, m2) implies m = m2
} by {
    if is_min(f, m) and is_min(f, m2) {
        is_min_apply(f, m)
        f(m)
        is_min_apply(f, m2)
        f(m2)
        is_min_false_below(f, m)
        false_below(f, m)
        is_min_false_below(f, m2)
        false_below(f, m2)
        if m2 < m {
            false_below_apply(f, m, m2)
            not f(m2)
            false
        }
        not (m2 < m)
        lt_or_lte(m2, m)
        m <= m2
        if m < m2 {
            false_below_apply(f, m2, m)
            not f(m)
            false
        }
        not (m < m2)
        lt_or_lte(m, m2)
        m2 <= m
        lte_antisymm(m, m2)
        m = m2
    }
}

// The hard direction of König's theorem.
//
// A minimum vertex cover of a bipartite graph has no more vertices than a maximum matching.
// The classical proofs run through augmenting paths (a maximum matching admits no augmenting
// path, and the vertices reachable from the unmatched side of a bipartition by alternating
// paths give a cover of the matching's size) or through Hall's marriage theorem. Neither
// machinery is available yet: the library has paths and walks, but no alternating-path
// toolkit, and the sufficiency direction of Hall's theorem is recorded but not proved in
// `graph/simple_graph_matching.ac`. The statement is recorded here for future work.
//
// theorem konig_hard_direction[V](g: SimpleGraph[V], s: FiniteSet[V]) {
//     is_bipartite(g, s) implies vertex_cover_number(g, s) <= maximum_matching_size(g, s)
// }

// König's theorem for bipartite graphs: the maximum matching size equals the minimum vertex
// cover size. The easy direction is proved above as
// `maximum_matching_size_le_vertex_cover_number`; the reverse direction is the hard
// direction recorded above.
//
// theorem konig_theorem[V](g: SimpleGraph[V], s: FiniteSet[V]) {
//     is_bipartite(g, s) implies maximum_matching_size(g, s) = vertex_cover_number(g, s)
// }

// Concrete cases below: the path P3, the cycle C4, and the complete bipartite graph K_{2,2}.
// Each graph is represented as the complete bipartite graph between the two sides of a
// two-colouring (P3 is K_{2,1} and C4 is K_{2,2}), which is exactly its bipartite
// structure. For each graph the maximum matching size and the minimum vertex cover size are
// computed directly, verifying König's equality on that instance.

/// Twice a natural number is at most three only below two.
theorem double_lte_three(n: Nat) {
    n + n <= Nat.3 implies n <= Nat.1
} by {
    if n + n <= Nat.3 {
        lt_or_lte(n, Nat.1)
        if n < Nat.1 {
            lt_suc_right(n, Nat.0)
            n = Nat.0 or n < Nat.0
            not_lt_zero(n)
            if n = Nat.0 {
                n <= Nat.1
            }
            n <= Nat.1
        }
        if Nat.1 <= n {
            if n = Nat.1 {
                n <= Nat.1
            }
            if n != Nat.1 {
                Nat.1 < n
                lt_imp_lte_suc(Nat.1, n)
                Nat.2 <= n
                lte_add_left(Nat.2, Nat.2, n)
                Nat.2 + Nat.2 <= Nat.2 + n
                lte_add_right(n, Nat.2, n)
                Nat.2 + n <= n + n
                lte_trans(Nat.4, Nat.2 + n, n + n)
                Nat.4 <= n + n
                lte_trans(Nat.4, n + n, Nat.3)
                Nat.4 <= Nat.3
                Nat.3.suc = Nat.4
                Nat.3 < Nat.3.suc
                Nat.3 < Nat.4
                lte_and_lt(Nat.4, Nat.3, Nat.4)
                Nat.4 < Nat.4
                lt_not_ref(Nat.4)
                false
            }
            n <= Nat.1
        }
        n <= Nat.1
    }
}

/// Twice a natural number is at most four only below three.
theorem double_lte_four(n: Nat) {
    n + n <= Nat.4 implies n <= Nat.2
} by {
    if n + n <= Nat.4 {
        lt_or_lte(n, Nat.2)
        if n < Nat.2 {
            lt_suc_right(n, Nat.1)
            n = Nat.1 or n < Nat.1
            if n = Nat.1 {
                n <= Nat.2
            }
            if not (n = Nat.1) {
                n < Nat.1
                lt_suc_right(n, Nat.0)
                n = Nat.0 or n < Nat.0
                not_lt_zero(n)
                n = Nat.0
                n <= Nat.2
            }
            n <= Nat.2
        }
        if Nat.2 <= n {
            if n = Nat.2 {
                n <= Nat.2
            }
            if n != Nat.2 {
                Nat.2 < n
                lt_imp_lte_suc(Nat.2, n)
                Nat.3 <= n
                lte_add_left(Nat.3, Nat.3, n)
                Nat.3 + Nat.3 <= Nat.3 + n
                lte_add_right(n, Nat.3, n)
                Nat.3 + n <= n + n
                lte_trans(Nat.6, Nat.3 + n, n + n)
                Nat.6 <= n + n
                lte_trans(Nat.6, n + n, Nat.4)
                Nat.6 <= Nat.4
                lt_suc(Nat.4)
                Nat.4 < Nat.5
                lt_imp_lt_suc(Nat.4, Nat.5)
                Nat.4 < Nat.6
                lte_and_lt(Nat.6, Nat.4, Nat.6)
                Nat.6 < Nat.6
                lt_not_ref(Nat.6)
                false
            }
            n <= Nat.2
        }
        n <= Nat.2
    }
}

// Generic tools for concrete matchings and vertex covers.

/// A single directed edge is a matching.
theorem single_pair_matching_is_matching[V](g: SimpleGraph[V], s: FiniteSet[V], a: Pair[V, V]) {
    directed_edges(g, s).contains(a) implies
        is_matching(g, s, fs_insert(FiniteSet.empty[Pair[V, V]], a))
} by {
    if directed_edges(g, s).contains(a) {
        forall(p: Pair[V, V]) {
            if fs_insert(FiniteSet.empty[Pair[V, V]], a).contains(p) {
                finite_set_singleton_contains_eq(a, p)
                fs_insert(FiniteSet.empty[Pair[V, V]], a).contains(p) = (p = a)
                p = a
                directed_edges(g, s).contains(p)
            }
            fs_insert(FiniteSet.empty[Pair[V, V]], a).contains(p) implies directed_edges(g, s).contains(p)
        }
        fs_subset_eq_intro(fs_insert(FiniteSet.empty[Pair[V, V]], a), directed_edges(g, s))
        fs_insert(FiniteSet.empty[Pair[V, V]], a).subset_eq(directed_edges(g, s))
        forall(p: Pair[V, V], q: Pair[V, V]) {
            if fs_insert(FiniteSet.empty[Pair[V, V]], a).contains(p) and fs_insert(FiniteSet.empty[Pair[V, V]], a).contains(q) and pairs_share_vertex(p, q) {
                finite_set_singleton_contains_eq(a, p)
                fs_insert(FiniteSet.empty[Pair[V, V]], a).contains(p) = (p = a)
                finite_set_singleton_contains_eq(a, q)
                fs_insert(FiniteSet.empty[Pair[V, V]], a).contains(q) = (q = a)
                p = a
                q = a
                p = q
            }
            fs_insert(FiniteSet.empty[Pair[V, V]], a).contains(p) and fs_insert(FiniteSet.empty[Pair[V, V]], a).contains(q) and pairs_share_vertex(p, q) implies p = q
        }
        is_matching_intro(g, s, fs_insert(FiniteSet.empty[Pair[V, V]], a))
        is_matching(g, s, fs_insert(FiniteSet.empty[Pair[V, V]], a))
    }
}

/// Sharing a vertex is symmetric.
theorem pairs_share_vertex_sym[V](p: Pair[V, V], q: Pair[V, V]) {
    pairs_share_vertex(p, q) = pairs_share_vertex(q, p)
} by {
    if pairs_share_vertex(p, q) {
        pairs_share_vertex(p, q) = (p.first = q.first or p.first = q.second or p.second = q.first or p.second = q.second)
        pairs_share_vertex(q, p) = (q.first = p.first or q.first = p.second or q.second = p.first or q.second = p.second)
        if p.first = q.first {
            pairs_share_vertex(q, p)
        }
        if not (p.first = q.first) {
            if p.first = q.second {
                pairs_share_vertex(q, p)
            }
            if not (p.first = q.second) {
                if p.second = q.first {
                    pairs_share_vertex(q, p)
                }
                if not (p.second = q.first) {
                    p.second = q.second
                    pairs_share_vertex(q, p)
                }
                pairs_share_vertex(q, p)
            }
            pairs_share_vertex(q, p)
        }
        pairs_share_vertex(q, p)
    }
    if not pairs_share_vertex(p, q) {
        if pairs_share_vertex(q, p) {
            pairs_share_vertex(q, p) = (q.first = p.first or q.first = p.second or q.second = p.first or q.second = p.second)
            pairs_share_vertex(p, q) = (p.first = q.first or p.first = q.second or p.second = q.first or p.second = q.second)
            if q.first = p.first {
                false
            }
            if not (q.first = p.first) {
                if q.first = p.second {
                    false
                }
                if not (q.first = p.second) {
                    if q.second = p.first {
                        false
                    }
                    if not (q.second = p.first) {
                        q.second = p.second
                        false
                    }
                    false
                }
                false
            }
            false
        }
        not pairs_share_vertex(q, p)
    }
    pairs_share_vertex(p, q) = pairs_share_vertex(q, p)
}

/// Equal pairs whose components do not share a vertex share nothing.
theorem share_false_of_eq[V](p: Pair[V, V], q: Pair[V, V], a: Pair[V, V], b: Pair[V, V]) {
    p = a and q = b and not pairs_share_vertex(a, b) implies pairs_share_vertex(p, q) = false
} by {
    if p = a and q = b and not pairs_share_vertex(a, b) {
        p = a
        q = b
        p.first = a.first
        q.first = b.first
        p.second = a.second
        q.second = b.second
        pairs_share_vertex(p, q) = pairs_share_vertex(a, b)
        pairs_share_vertex(a, b) = false
        pairs_share_vertex(p, q) = false
    }
}

/// Two directed edges that share no vertex form a matching.
///
/// The pair of `a` with itself, `a` with `b`, `b` with `a`, and `b` with itself are the only
/// ways to choose two members; the mixed cases cannot happen because the two pairs share no
/// vertex, so the matching condition holds vacuously there.
theorem two_pair_matching_is_matching[V](g: SimpleGraph[V], s: FiniteSet[V], a: Pair[V, V], b: Pair[V, V]) {
    directed_edges(g, s).contains(a) and directed_edges(g, s).contains(b) and
        not pairs_share_vertex(a, b)
        implies is_matching(g, s, fs_insert(fs_insert(FiniteSet.empty[Pair[V, V]], a), b))
} by {
    if directed_edges(g, s).contains(a) and directed_edges(g, s).contains(b) and
        not pairs_share_vertex(a, b) {
        forall(p: Pair[V, V]) {
            if fs_insert(fs_insert(FiniteSet.empty[Pair[V, V]], a), b).contains(p) {
                fs_insert_contains_eq(fs_insert(FiniteSet.empty[Pair[V, V]], a), b, p)
                finite_set_singleton_contains_eq(a, p)
                fs_insert(fs_insert(FiniteSet.empty[Pair[V, V]], a), b).contains(p) = (p = b or (p = a))
                if p = a {
                    directed_edges(g, s).contains(p)
                }
                if not (p = a) {
                    p = b
                    directed_edges(g, s).contains(p)
                }
                directed_edges(g, s).contains(p)
            }
            fs_insert(fs_insert(FiniteSet.empty[Pair[V, V]], a), b).contains(p) implies directed_edges(g, s).contains(p)
        }
        fs_subset_eq_intro(fs_insert(fs_insert(FiniteSet.empty[Pair[V, V]], a), b), directed_edges(g, s))
        fs_insert(fs_insert(FiniteSet.empty[Pair[V, V]], a), b).subset_eq(directed_edges(g, s))
        forall(p: Pair[V, V], q: Pair[V, V]) {
            fs_insert_contains_eq(fs_insert(FiniteSet.empty[Pair[V, V]], a), b, p)
            finite_set_singleton_contains_eq(a, p)
            fs_insert_contains_eq(fs_insert(FiniteSet.empty[Pair[V, V]], a), b, q)
            finite_set_singleton_contains_eq(a, q)
            fs_insert(fs_insert(FiniteSet.empty[Pair[V, V]], a), b).contains(p) = (p = b or (p = a))
            fs_insert(fs_insert(FiniteSet.empty[Pair[V, V]], a), b).contains(q) = (q = b or (q = a))
            if p = a {
                if q = a {
                    if fs_insert(fs_insert(FiniteSet.empty[Pair[V, V]], a), b).contains(p) and fs_insert(fs_insert(FiniteSet.empty[Pair[V, V]], a), b).contains(q) and pairs_share_vertex(p, q) {
                        p = q
                    }
                    fs_insert(fs_insert(FiniteSet.empty[Pair[V, V]], a), b).contains(p) and fs_insert(fs_insert(FiniteSet.empty[Pair[V, V]], a), b).contains(q) and pairs_share_vertex(p, q) implies p = q
                }
                if not (q = a) {
                    if q = b {
                        p = a
                        q = b
                        share_false_of_eq(p, q, a, b)
                        pairs_share_vertex(p, q) = false
                        (fs_insert(fs_insert(FiniteSet.empty[Pair[V, V]], a), b).contains(p) and fs_insert(fs_insert(FiniteSet.empty[Pair[V, V]], a), b).contains(q) and pairs_share_vertex(p, q)) = false
                        false_implies(p = q)
                        (false implies p = q) = true
                        ((fs_insert(fs_insert(FiniteSet.empty[Pair[V, V]], a), b).contains(p) and fs_insert(fs_insert(FiniteSet.empty[Pair[V, V]], a), b).contains(q) and pairs_share_vertex(p, q)) implies p = q) = true
                        fs_insert(fs_insert(FiniteSet.empty[Pair[V, V]], a), b).contains(p) and fs_insert(fs_insert(FiniteSet.empty[Pair[V, V]], a), b).contains(q) and pairs_share_vertex(p, q) implies p = q
                    }
                    if not (q = b) {
                        fs_insert(fs_insert(FiniteSet.empty[Pair[V, V]], a), b).contains(q) = (q = b or (q = a))
                        fs_insert(fs_insert(FiniteSet.empty[Pair[V, V]], a), b).contains(q) = false
                        (fs_insert(fs_insert(FiniteSet.empty[Pair[V, V]], a), b).contains(p) and fs_insert(fs_insert(FiniteSet.empty[Pair[V, V]], a), b).contains(q) and pairs_share_vertex(p, q)) = false
                        false_implies(p = q)
                        (false implies p = q) = true
                        ((fs_insert(fs_insert(FiniteSet.empty[Pair[V, V]], a), b).contains(p) and fs_insert(fs_insert(FiniteSet.empty[Pair[V, V]], a), b).contains(q) and pairs_share_vertex(p, q)) implies p = q) = true
                        fs_insert(fs_insert(FiniteSet.empty[Pair[V, V]], a), b).contains(p) and fs_insert(fs_insert(FiniteSet.empty[Pair[V, V]], a), b).contains(q) and pairs_share_vertex(p, q) implies p = q
                    }
                    fs_insert(fs_insert(FiniteSet.empty[Pair[V, V]], a), b).contains(p) and fs_insert(fs_insert(FiniteSet.empty[Pair[V, V]], a), b).contains(q) and pairs_share_vertex(p, q) implies p = q
                }
                fs_insert(fs_insert(FiniteSet.empty[Pair[V, V]], a), b).contains(p) and fs_insert(fs_insert(FiniteSet.empty[Pair[V, V]], a), b).contains(q) and pairs_share_vertex(p, q) implies p = q
            }
            if not (p = a) {
                if p = b {
                    if q = b {
                        if fs_insert(fs_insert(FiniteSet.empty[Pair[V, V]], a), b).contains(p) and fs_insert(fs_insert(FiniteSet.empty[Pair[V, V]], a), b).contains(q) and pairs_share_vertex(p, q) {
                            p = q
                        }
                        fs_insert(fs_insert(FiniteSet.empty[Pair[V, V]], a), b).contains(p) and fs_insert(fs_insert(FiniteSet.empty[Pair[V, V]], a), b).contains(q) and pairs_share_vertex(p, q) implies p = q
                    }
                    if not (q = b) {
                        if q = a {
                            p = b
                            q = a
                            pairs_share_vertex_sym(a, b)
                            pairs_share_vertex(a, b) = pairs_share_vertex(b, a)
                            share_false_of_eq(p, q, b, a)
                            pairs_share_vertex(p, q) = false
                            (fs_insert(fs_insert(FiniteSet.empty[Pair[V, V]], a), b).contains(p) and fs_insert(fs_insert(FiniteSet.empty[Pair[V, V]], a), b).contains(q) and pairs_share_vertex(p, q)) = false
                            false_implies(p = q)
                            (false implies p = q) = true
                            ((fs_insert(fs_insert(FiniteSet.empty[Pair[V, V]], a), b).contains(p) and fs_insert(fs_insert(FiniteSet.empty[Pair[V, V]], a), b).contains(q) and pairs_share_vertex(p, q)) implies p = q) = true
                            fs_insert(fs_insert(FiniteSet.empty[Pair[V, V]], a), b).contains(p) and fs_insert(fs_insert(FiniteSet.empty[Pair[V, V]], a), b).contains(q) and pairs_share_vertex(p, q) implies p = q
                        }
                        if not (q = a) {
                            fs_insert(fs_insert(FiniteSet.empty[Pair[V, V]], a), b).contains(q) = false
                            (fs_insert(fs_insert(FiniteSet.empty[Pair[V, V]], a), b).contains(p) and fs_insert(fs_insert(FiniteSet.empty[Pair[V, V]], a), b).contains(q) and pairs_share_vertex(p, q)) = false
                            false_implies(p = q)
                            (false implies p = q) = true
                            ((fs_insert(fs_insert(FiniteSet.empty[Pair[V, V]], a), b).contains(p) and fs_insert(fs_insert(FiniteSet.empty[Pair[V, V]], a), b).contains(q) and pairs_share_vertex(p, q)) implies p = q) = true
                            fs_insert(fs_insert(FiniteSet.empty[Pair[V, V]], a), b).contains(p) and fs_insert(fs_insert(FiniteSet.empty[Pair[V, V]], a), b).contains(q) and pairs_share_vertex(p, q) implies p = q
                        }
                        fs_insert(fs_insert(FiniteSet.empty[Pair[V, V]], a), b).contains(p) and fs_insert(fs_insert(FiniteSet.empty[Pair[V, V]], a), b).contains(q) and pairs_share_vertex(p, q) implies p = q
                    }
                    fs_insert(fs_insert(FiniteSet.empty[Pair[V, V]], a), b).contains(p) and fs_insert(fs_insert(FiniteSet.empty[Pair[V, V]], a), b).contains(q) and pairs_share_vertex(p, q) implies p = q
                }
                if not (p = b) {
                    fs_insert(fs_insert(FiniteSet.empty[Pair[V, V]], a), b).contains(p) = false
                    (fs_insert(fs_insert(FiniteSet.empty[Pair[V, V]], a), b).contains(p) and fs_insert(fs_insert(FiniteSet.empty[Pair[V, V]], a), b).contains(q) and pairs_share_vertex(p, q)) = false
                    false_implies(p = q)
                    (false implies p = q) = true
                    ((fs_insert(fs_insert(FiniteSet.empty[Pair[V, V]], a), b).contains(p) and fs_insert(fs_insert(FiniteSet.empty[Pair[V, V]], a), b).contains(q) and pairs_share_vertex(p, q)) implies p = q) = true
                    fs_insert(fs_insert(FiniteSet.empty[Pair[V, V]], a), b).contains(p) and fs_insert(fs_insert(FiniteSet.empty[Pair[V, V]], a), b).contains(q) and pairs_share_vertex(p, q) implies p = q
                }
                fs_insert(fs_insert(FiniteSet.empty[Pair[V, V]], a), b).contains(p) and fs_insert(fs_insert(FiniteSet.empty[Pair[V, V]], a), b).contains(q) and pairs_share_vertex(p, q) implies p = q
            }
            fs_insert(fs_insert(FiniteSet.empty[Pair[V, V]], a), b).contains(p) and fs_insert(fs_insert(FiniteSet.empty[Pair[V, V]], a), b).contains(q) and pairs_share_vertex(p, q) implies p = q
        }
        is_matching_intro(g, s, fs_insert(fs_insert(FiniteSet.empty[Pair[V, V]], a), b))
        is_matching(g, s, fs_insert(fs_insert(FiniteSet.empty[Pair[V, V]], a), b))
    }
}

/// Pairs with pairwise-distinct components share no vertex.
theorem pairs_not_share_of_ne_components[V](a: Pair[V, V], b: Pair[V, V]) {
    a.first != b.first and a.first != b.second and a.second != b.first and a.second != b.second
        implies not pairs_share_vertex(a, b)
} by {
    if a.first != b.first and a.first != b.second and a.second != b.first and a.second != b.second {
        pairs_share_vertex(a, b) = (a.first = b.first or a.first = b.second or a.second = b.first or a.second = b.second)
        if a.first = b.first or a.first = b.second or a.second = b.first or a.second = b.second {
            if a.first = b.first {
                false
            }
            if not (a.first = b.first) {
                if a.first = b.second {
                    false
                }
                if not (a.first = b.second) {
                    if a.second = b.first {
                        false
                    }
                    if not (a.second = b.first) {
                        a.second = b.second
                        false
                    }
                    false
                }
                false
            }
            false
        }
        not pairs_share_vertex(a, b)
    }
}

/// A value equal to neither of two others is neither of them.
theorem part_false_of_ne[V](v: V, a: V, b: V) {
    v != a and v != b implies (v = a or v = b) = false
} by {
    if v != a and v != b {
        if v = a or v = b {
            if v = a {
                false
            }
            if not (v = a) {
                v = b
                false
            }
            false
        }
        not (v = a or v = b)
        (v = a or v = b) = false
    }
}

// --- The path P3: vertices 0, 1, 2 with the single edge set {0,1} and {1,2}. ---
//
// The path on three vertices is the complete bipartite graph with the middle vertex on one
// side and the two ends on the other, so its adjacency is `part(x) != part(y)` for the
// colouring that picks out the middle vertex.

/// The two-colouring of the path: the middle vertex alone is one side.
define path3_part(v: Nat) -> Bool {
    v = Nat.1
}

/// Adjacency of the path: one endpoint is the middle vertex, the other is not.
define path3_adj(x: Nat, y: Nat) -> Bool {
    path3_part(x) != path3_part(y)
}

/// The path adjacency relation is symmetric.
theorem path3_adj_is_symmetric {
    is_symmetric(path3_adj)
} by {
    forall(x: Nat, y: Nat) {
        if path3_adj(x, y) {
            path3_adj(x, y) = (path3_part(x) != path3_part(y))
            path3_part(x) != path3_part(y)
            path3_part(y) != path3_part(x)
            path3_adj(y, x) = (path3_part(y) != path3_part(x))
            path3_adj(y, x)
        }
        path3_adj(x, y) implies path3_adj(y, x)
    }
}

/// The path adjacency relation has no loops.
theorem path3_adj_is_irreflexive {
    is_irreflexive(path3_adj)
} by {
    forall(x: Nat) {
        if path3_adj(x, x) {
            path3_adj(x, x) = (path3_part(x) != path3_part(x))
            false
        }
        not path3_adj(x, x)
    }
}

/// The path adjacency relation satisfies the simple-graph constraint.
theorem path3_adj_constraint {
    is_symmetric(path3_adj) and is_irreflexive(path3_adj)
} by {
    path3_adj_is_symmetric
    path3_adj_is_irreflexive
}

/// The path graph on three vertices exists.
theorem path3_graph_exists {
    exists(g: SimpleGraph[Nat]) { SimpleGraph.new(path3_adj) = Option.some(g) }
} by {
    path3_adj_constraint
}

/// The path graph on three vertices.
let path3: SimpleGraph[Nat] satisfy {
    SimpleGraph.new(path3_adj) = Option.some(path3)
}

/// Adjacency in the path graph is the path adjacency relation.
theorem path3_adj_eq(x: Nat, y: Nat) {
    path3.adj(x, y) = (path3_part(x) != path3_part(y))
} by {
    path3.adj = path3_adj
    path3.adj(x, y) = path3_adj(x, y)
    path3_adj(x, y) = (path3_part(x) != path3_part(y))
}

/// The path P3 is bipartite.
theorem path3_is_bipartition {
    is_bipartition(path3, range_set(Nat.3), path3_part)
} by {
    forall(x: Nat, y: Nat) {
        if range_set(Nat.3).contains(x) and range_set(Nat.3).contains(y) and path3.adj(x, y) {
            path3_adj_eq(x, y)
            path3_part(x) != path3_part(y)
        }
        (range_set(Nat.3).contains(x) and range_set(Nat.3).contains(y) and path3.adj(x, y)) implies path3_part(x) != path3_part(y)
    }
    is_bipartition_intro(path3, range_set(Nat.3), path3_part)
    is_bipartition(path3, range_set(Nat.3), path3_part)
}

/// The path P3 is bipartite.
theorem path3_is_bipartite {
    is_bipartite(path3, range_set(Nat.3))
} by {
    path3_is_bipartition
    is_bipartite_intro(path3, range_set(Nat.3), path3_part)
    is_bipartite(path3, range_set(Nat.3))
}

/// The vertices of the path: `0` and `1` lie below `3`.
theorem path3_vertices_lt {
    Nat.0 < Nat.3 and Nat.1 < Nat.3
} by {
    Nat.0.suc = Nat.1
    Nat.0 < Nat.0.suc
    Nat.0 < Nat.1
    Nat.1.suc = Nat.2
    Nat.1 < Nat.1.suc
    Nat.1 < Nat.2
    Nat.2.suc = Nat.3
    Nat.2 < Nat.2.suc
    Nat.2 < Nat.3
    Nat.1 < Nat.2
    Nat.2 <= Nat.3
    lt_and_lte(Nat.1, Nat.2, Nat.3)
    Nat.1 < Nat.3
    Nat.0 < Nat.1
    Nat.1 <= Nat.3
    lt_and_lte(Nat.0, Nat.1, Nat.3)
    Nat.0 < Nat.3
}

/// Vertex `0` lies in the vertex set of the path.
theorem path3_vertex_zero {
    range_set(Nat.3).contains(Nat.0)
} by {
    path3_vertices_lt
    Nat.0 < Nat.3
    range_set_contains(Nat.3, Nat.0)
    range_set(Nat.3).contains(Nat.0)
}

/// Vertex `1` lies in the vertex set of the path.
theorem path3_vertex_one {
    range_set(Nat.3).contains(Nat.1)
} by {
    path3_vertices_lt
    Nat.1 < Nat.3
    range_set_contains(Nat.3, Nat.1)
    range_set(Nat.3).contains(Nat.1)
}

/// The edge `0`--`1` is present in the path.
theorem path3_edge01 {
    path3.adj(Nat.0, Nat.1)
} by {
    path3_adj_eq(Nat.0, Nat.1)
    path3.adj(Nat.0, Nat.1) = (path3_part(Nat.0) != path3_part(Nat.1))
    path3_part(Nat.0) = (Nat.0 = Nat.1)
    path3_part(Nat.1) = (Nat.1 = Nat.1)
    path3_part(Nat.0) != path3_part(Nat.1)
    path3.adj(Nat.0, Nat.1)
}

/// The edge `0`--`1` is a directed edge of the path.
theorem path3_directed_edge01 {
    directed_edges(path3, range_set(Nat.3)).contains(Pair.new(Nat.0, Nat.1))
} by {
    path3_vertex_zero
    range_set(Nat.3).contains(Nat.0)
    path3_vertex_one
    range_set(Nat.3).contains(Nat.1)
    path3_edge01
    path3.adj(Nat.0, Nat.1)
    directed_edges_contains_pair(path3, range_set(Nat.3), Nat.0, Nat.1)
    directed_edges(path3, range_set(Nat.3)).contains(Pair.new(Nat.0, Nat.1))
}

/// The size-one matching of the path: the edge {0, 1}.
let path3_matching: FiniteSet[Pair[Nat, Nat]] = fs_insert(FiniteSet.empty[Pair[Nat, Nat]], Pair.new(Nat.0, Nat.1))

/// The one-edge set of the path is a matching.
theorem path3_matching_is_matching {
    is_matching(path3, range_set(Nat.3), path3_matching)
} by {
    path3_directed_edge01
    directed_edges(path3, range_set(Nat.3)).contains(Pair.new(Nat.0, Nat.1))
    single_pair_matching_is_matching(path3, range_set(Nat.3), Pair.new(Nat.0, Nat.1))
    is_matching(path3, range_set(Nat.3), fs_insert(FiniteSet.empty[Pair[Nat, Nat]], Pair.new(Nat.0, Nat.1)))
    path3_matching = fs_insert(FiniteSet.empty[Pair[Nat, Nat]], Pair.new(Nat.0, Nat.1))
    is_matching(path3, range_set(Nat.3), path3_matching)
}

/// The one-edge matching of the path has one pair.
theorem path3_matching_card {
    fs_card(path3_matching) = Nat.1
} by {
    fs_card_singleton(Pair.new(Nat.0, Nat.1))
    fs_card(fs_insert(FiniteSet.empty[Pair[Nat, Nat]], Pair.new(Nat.0, Nat.1))) = Nat.1
    path3_matching = fs_insert(FiniteSet.empty[Pair[Nat, Nat]], Pair.new(Nat.0, Nat.1))
    fs_card(path3_matching) = Nat.1
}

/// No matching of the path is larger than one edge.
theorem path3_matching_size_at_most_one(m: FiniteSet[Pair[Nat, Nat]]) {
    is_matching(path3, range_set(Nat.3), m) implies fs_card(m) <= Nat.1
} by {
    if is_matching(path3, range_set(Nat.3), m) {
        matching_size_le_half_vertices(path3, range_set(Nat.3), m)
        fs_card(m) + fs_card(m) <= fs_card(range_set(Nat.3))
        range_set_card(Nat.3)
        fs_card(range_set(Nat.3)) = Nat.3
        fs_card(m) + fs_card(m) <= Nat.3
        double_lte_three(fs_card(m))
        fs_card(m) <= Nat.1
    }
}

/// One is an achieved matching size of the path.
theorem path3_matching_size_pred_one {
    matching_size_pred(path3, range_set(Nat.3))(Nat.1)
} by {
    path3_matching_is_matching
    is_matching(path3, range_set(Nat.3), path3_matching)
    path3_matching_card
    fs_card(path3_matching) = Nat.1
    matching_size_pred_intro(path3, range_set(Nat.3), path3_matching)
    matching_size_pred(path3, range_set(Nat.3))(fs_card(path3_matching))
    matching_size_pred(path3, range_set(Nat.3))(Nat.1)
}

/// Every achieved matching size of the path is at most one.
theorem path3_matching_size_pred_bounded(n: Nat) {
    matching_size_pred(path3, range_set(Nat.3))(n) implies n <= Nat.1
} by {
    if matching_size_pred(path3, range_set(Nat.3))(n) {
        matching_size_pred(path3, range_set(Nat.3))(n) = exists(m: FiniteSet[Pair[Nat, Nat]]) {
            is_matching(path3, range_set(Nat.3), m) and fs_card(m) = n
        }
        let m: FiniteSet[Pair[Nat, Nat]] satisfy {
            is_matching(path3, range_set(Nat.3), m) and fs_card(m) = n
        }
        path3_matching_size_at_most_one(m)
        fs_card(m) <= Nat.1
        fs_card(m) = n
        n <= Nat.1
    }
}

/// The maximum matching size of the path is one.
theorem path3_maximum_matching_size {
    maximum_matching_size(path3, range_set(Nat.3)) = Nat.1
} by {
    path3_matching_size_pred_one
    matching_size_pred(path3, range_set(Nat.3))(Nat.1)
    forall(n: Nat) {
        path3_matching_size_pred_bounded(n)
        matching_size_pred(path3, range_set(Nat.3))(n) implies n <= Nat.1
    }
    is_upper_bound_of_intro(matching_size_pred(path3, range_set(Nat.3)), Nat.1)
    is_upper_bound_of(matching_size_pred(path3, range_set(Nat.3)), Nat.1)
    is_max_intro(matching_size_pred(path3, range_set(Nat.3)), Nat.1)
    is_max(matching_size_pred(path3, range_set(Nat.3)), Nat.1)
    is_max(matching_size_pred(path3, range_set(Nat.3)), maximum_matching_size(path3, range_set(Nat.3)))
    max_unique(matching_size_pred(path3, range_set(Nat.3)), Nat.1, maximum_matching_size(path3, range_set(Nat.3)))
    maximum_matching_size(path3, range_set(Nat.3)) = Nat.1
}

/// The size-one vertex cover of the path: the middle vertex.
let path3_cover: FiniteSet[Nat] = fs_insert(FiniteSet.empty[Nat], Nat.1)

/// An edge of the path meets the middle vertex.
theorem path3_adj_covered(x: Nat, y: Nat) {
    path3.adj(x, y) implies path3_cover.contains(x) or path3_cover.contains(y)
} by {
    if path3.adj(x, y) {
        path3_adj_eq(x, y)
        path3_part(x) != path3_part(y)
        path3_part(x) = (x = Nat.1)
        path3_part(y) = (y = Nat.1)
        finite_set_singleton_contains_eq(Nat.1, x)
        finite_set_singleton_contains_eq(Nat.1, y)
        path3_cover.contains(x) = (x = Nat.1)
        path3_cover.contains(y) = (y = Nat.1)
        if path3_part(x) {
            x = Nat.1
            path3_cover.contains(x)
            path3_cover.contains(x) or path3_cover.contains(y)
        }
        if not path3_part(x) {
            path3_part(y)
            y = Nat.1
            path3_cover.contains(y)
            path3_cover.contains(x) or path3_cover.contains(y)
        }
        path3_cover.contains(x) or path3_cover.contains(y)
    }
}

/// The middle vertex covers every edge of the path.
theorem path3_cover_is_vertex_cover {
    is_vertex_cover(path3, range_set(Nat.3), path3_cover)
} by {
    forall(x: Nat, y: Nat) {
        if range_set(Nat.3).contains(x) and range_set(Nat.3).contains(y) and path3.adj(x, y) {
            path3_adj_covered(x, y)
            path3_cover.contains(x) or path3_cover.contains(y)
        }
        (range_set(Nat.3).contains(x) and range_set(Nat.3).contains(y) and path3.adj(x, y)) implies path3_cover.contains(x) or path3_cover.contains(y)
    }
    is_vertex_cover_intro(path3, range_set(Nat.3), path3_cover)
    is_vertex_cover(path3, range_set(Nat.3), path3_cover)
}

/// The middle vertex lies in the vertex set of the path.
theorem path3_cover_subset {
    path3_cover.subset_eq(range_set(Nat.3))
} by {
    path3_vertices_lt
    Nat.1 < Nat.3
    range_set_contains(Nat.3, Nat.1)
    range_set(Nat.3).contains(Nat.1)
    finite_set_singleton_subset_of_contains(range_set(Nat.3), Nat.1)
    fs_insert(FiniteSet.empty[Nat], Nat.1).subset_eq(range_set(Nat.3))
    path3_cover = fs_insert(FiniteSet.empty[Nat], Nat.1)
    path3_cover.subset_eq(range_set(Nat.3))
}

/// The size-one cover of the path has one vertex.
theorem path3_cover_card {
    fs_card(path3_cover) = Nat.1
} by {
    fs_card_singleton(Nat.1)
    fs_card(fs_insert(FiniteSet.empty[Nat], Nat.1)) = Nat.1
    path3_cover = fs_insert(FiniteSet.empty[Nat], Nat.1)
    fs_card(path3_cover) = Nat.1
}

/// One is an achieved cover size of the path.
theorem path3_cover_size_pred_one {
    cover_size_pred(path3, range_set(Nat.3))(Nat.1)
} by {
    path3_cover_subset
    path3_cover.subset_eq(range_set(Nat.3))
    path3_cover_is_vertex_cover
    is_vertex_cover(path3, range_set(Nat.3), path3_cover)
    path3_cover_card
    fs_card(path3_cover) = Nat.1
    cover_size_pred_intro(path3, range_set(Nat.3), path3_cover)
    cover_size_pred(path3, range_set(Nat.3))(fs_card(path3_cover))
    cover_size_pred(path3, range_set(Nat.3))(Nat.1)
}

/// Every achieved cover size of the path is at least one.
theorem path3_cover_size_pred_bounded(n: Nat) {
    cover_size_pred(path3, range_set(Nat.3))(n) implies Nat.1 <= n
} by {
    if cover_size_pred(path3, range_set(Nat.3))(n) {
        cover_size_pred(path3, range_set(Nat.3))(n) = exists(c: FiniteSet[Nat]) {
            c.subset_eq(range_set(Nat.3)) and is_vertex_cover(path3, range_set(Nat.3), c) and fs_card(c) = n
        }
        let c: FiniteSet[Nat] satisfy {
            c.subset_eq(range_set(Nat.3)) and is_vertex_cover(path3, range_set(Nat.3), c) and fs_card(c) = n
        }
        path3_matching_is_matching
        is_matching(path3, range_set(Nat.3), path3_matching)
        vertex_cover_bounds_matching(path3, range_set(Nat.3), c, path3_matching)
        fs_card(path3_matching) <= fs_card(c)
        path3_matching_card
        fs_card(path3_matching) = Nat.1
        Nat.1 <= fs_card(c)
        fs_card(c) = n
        Nat.1 <= n
    }
}

/// The minimum vertex cover number of the path is one.
theorem path3_vertex_cover_number {
    vertex_cover_number(path3, range_set(Nat.3)) = Nat.1
} by {
    path3_cover_size_pred_one
    cover_size_pred(path3, range_set(Nat.3))(Nat.1)
    forall(x: Nat) {
        if x < Nat.1 {
            if cover_size_pred(path3, range_set(Nat.3))(x) {
                path3_cover_size_pred_bounded(x)
                Nat.1 <= x
                lte_and_lt(Nat.1, x, Nat.1)
                Nat.1 < Nat.1
                lt_not_ref(Nat.1)
                false
            }
            not cover_size_pred(path3, range_set(Nat.3))(x)
        }
        x < Nat.1 implies not cover_size_pred(path3, range_set(Nat.3))(x)
    }
    false_below(cover_size_pred(path3, range_set(Nat.3)), Nat.1)
    is_min(cover_size_pred(path3, range_set(Nat.3)), Nat.1)
    is_min(cover_size_pred(path3, range_set(Nat.3)), vertex_cover_number(path3, range_set(Nat.3)))
    min_unique(cover_size_pred(path3, range_set(Nat.3)), Nat.1, vertex_cover_number(path3, range_set(Nat.3)))
    vertex_cover_number(path3, range_set(Nat.3)) = Nat.1
}

/// König's equality on the path P3.
theorem konig_on_path3 {
    is_bipartite(path3, range_set(Nat.3)) and
        maximum_matching_size(path3, range_set(Nat.3)) = vertex_cover_number(path3, range_set(Nat.3))
} by {
    path3_is_bipartite
    is_bipartite(path3, range_set(Nat.3))
    path3_maximum_matching_size
    maximum_matching_size(path3, range_set(Nat.3)) = Nat.1
    path3_vertex_cover_number
    vertex_cover_number(path3, range_set(Nat.3)) = Nat.1
    maximum_matching_size(path3, range_set(Nat.3)) = vertex_cover_number(path3, range_set(Nat.3))
}

// --- The cycle C4: vertices 0, 1, 2, 3 with edges 0-1, 1-2, 2-3, 3-0. ---
//
// The cycle on four vertices is the complete bipartite graph with the sides {0, 2} and
// {1, 3}, so its adjacency is `part(x) != part(y)` for the colouring that picks out the
// even vertices.

/// The two-colouring of the cycle: the even vertices form one side.
define c4_part(v: Nat) -> Bool {
    v = Nat.0 or v = Nat.2
}

/// Adjacency of the cycle: the two endpoints lie on opposite sides.
define c4_adj(x: Nat, y: Nat) -> Bool {
    c4_part(x) != c4_part(y)
}

/// The cycle adjacency relation is symmetric.
theorem c4_adj_is_symmetric {
    is_symmetric(c4_adj)
} by {
    forall(x: Nat, y: Nat) {
        if c4_adj(x, y) {
            c4_adj(x, y) = (c4_part(x) != c4_part(y))
            c4_part(x) != c4_part(y)
            c4_part(y) != c4_part(x)
            c4_adj(y, x) = (c4_part(y) != c4_part(x))
            c4_adj(y, x)
        }
        c4_adj(x, y) implies c4_adj(y, x)
    }
}

/// The cycle adjacency relation has no loops.
theorem c4_adj_is_irreflexive {
    is_irreflexive(c4_adj)
} by {
    forall(x: Nat) {
        if c4_adj(x, x) {
            c4_adj(x, x) = (c4_part(x) != c4_part(x))
            false
        }
        not c4_adj(x, x)
    }
}

/// The cycle adjacency relation satisfies the simple-graph constraint.
theorem c4_adj_constraint {
    is_symmetric(c4_adj) and is_irreflexive(c4_adj)
} by {
    c4_adj_is_symmetric
    c4_adj_is_irreflexive
}

/// The cycle graph on four vertices exists.
theorem c4_graph_exists {
    exists(g: SimpleGraph[Nat]) { SimpleGraph.new(c4_adj) = Option.some(g) }
} by {
    c4_adj_constraint
}

/// The cycle graph on four vertices.
let c4: SimpleGraph[Nat] satisfy {
    SimpleGraph.new(c4_adj) = Option.some(c4)
}

/// Adjacency in the cycle graph is the cycle adjacency relation.
theorem c4_adj_eq(x: Nat, y: Nat) {
    c4.adj(x, y) = (c4_part(x) != c4_part(y))
} by {
    c4.adj = c4_adj
    c4.adj(x, y) = c4_adj(x, y)
    c4_adj(x, y) = (c4_part(x) != c4_part(y))
}

/// The cycle C4 is bipartite.
theorem c4_is_bipartition {
    is_bipartition(c4, range_set(Nat.4), c4_part)
} by {
    forall(x: Nat, y: Nat) {
        if range_set(Nat.4).contains(x) and range_set(Nat.4).contains(y) and c4.adj(x, y) {
            c4_adj_eq(x, y)
            c4_part(x) != c4_part(y)
        }
        (range_set(Nat.4).contains(x) and range_set(Nat.4).contains(y) and c4.adj(x, y)) implies c4_part(x) != c4_part(y)
    }
    is_bipartition_intro(c4, range_set(Nat.4), c4_part)
    is_bipartition(c4, range_set(Nat.4), c4_part)
}

/// The cycle C4 is bipartite.
theorem c4_is_bipartite {
    is_bipartite(c4, range_set(Nat.4))
} by {
    c4_is_bipartition
    is_bipartite_intro(c4, range_set(Nat.4), c4_part)
    is_bipartite(c4, range_set(Nat.4))
}

/// The vertices of the cycle: `0`, `1`, `2`, `3` lie below `4`.
theorem c4_vertices_lt {
    Nat.0 < Nat.4 and Nat.1 < Nat.4 and Nat.2 < Nat.4 and Nat.3 < Nat.4
} by {
    Nat.0.suc = Nat.1
    Nat.0 < Nat.0.suc
    Nat.0 < Nat.1
    Nat.1.suc = Nat.2
    Nat.1 < Nat.1.suc
    Nat.1 < Nat.2
    Nat.2.suc = Nat.3
    Nat.2 < Nat.2.suc
    Nat.2 < Nat.3
    Nat.3.suc = Nat.4
    Nat.3 < Nat.3.suc
    Nat.3 < Nat.4
    Nat.2 < Nat.3
    Nat.3 <= Nat.4
    lt_and_lte(Nat.2, Nat.3, Nat.4)
    Nat.2 < Nat.4
    Nat.1 < Nat.2
    Nat.2 <= Nat.4
    lt_and_lte(Nat.1, Nat.2, Nat.4)
    Nat.1 < Nat.4
    Nat.0 < Nat.1
    Nat.1 <= Nat.4
    lt_and_lte(Nat.0, Nat.1, Nat.4)
    Nat.0 < Nat.4
}

/// Vertex `0` lies in the vertex set of the cycle.
theorem c4_vertex0 {
    range_set(Nat.4).contains(Nat.0)
} by {
    c4_vertices_lt
    Nat.0 < Nat.4
    range_set_contains(Nat.4, Nat.0)
    range_set(Nat.4).contains(Nat.0)
}

/// Vertex `1` lies in the vertex set of the cycle.
theorem c4_vertex1 {
    range_set(Nat.4).contains(Nat.1)
} by {
    c4_vertices_lt
    Nat.1 < Nat.4
    range_set_contains(Nat.4, Nat.1)
    range_set(Nat.4).contains(Nat.1)
}

/// Vertex `2` lies in the vertex set of the cycle.
theorem c4_vertex2 {
    range_set(Nat.4).contains(Nat.2)
} by {
    c4_vertices_lt
    Nat.2 < Nat.4
    range_set_contains(Nat.4, Nat.2)
    range_set(Nat.4).contains(Nat.2)
}

/// Vertex `3` lies in the vertex set of the cycle.
theorem c4_vertex3 {
    range_set(Nat.4).contains(Nat.3)
} by {
    c4_vertices_lt
    Nat.3 < Nat.4
    range_set_contains(Nat.4, Nat.3)
    range_set(Nat.4).contains(Nat.3)
}

/// The colour of `0` is true.
theorem c4_part0 {
    c4_part(Nat.0) = true
} by {
    c4_part(Nat.0) = (Nat.0 = Nat.0 or Nat.0 = Nat.2)
    Nat.0 = Nat.0
    c4_part(Nat.0) = true
}

/// The colour of `1` is false.
theorem c4_part1 {
    c4_part(Nat.1) = false
} by {
    c4_part(Nat.1) = (Nat.1 = Nat.0 or Nat.1 = Nat.2)
    Nat.1 != Nat.0
    Nat.1 != Nat.2
    part_false_of_ne(Nat.1, Nat.0, Nat.2)
    (Nat.1 = Nat.0 or Nat.1 = Nat.2) = false
    c4_part(Nat.1) = false
}

/// The colour of `2` is true.
theorem c4_part2 {
    c4_part(Nat.2) = true
} by {
    c4_part(Nat.2) = (Nat.2 = Nat.0 or Nat.2 = Nat.2)
    Nat.2 = Nat.2
    c4_part(Nat.2) = true
}

/// The colour of `3` is false.
theorem c4_part3 {
    c4_part(Nat.3) = false
} by {
    c4_part(Nat.3) = (Nat.3 = Nat.0 or Nat.3 = Nat.2)
    Nat.3 != Nat.0
    Nat.3 != Nat.2
    part_false_of_ne(Nat.3, Nat.0, Nat.2)
    (Nat.3 = Nat.0 or Nat.3 = Nat.2) = false
    c4_part(Nat.3) = false
}

/// The edge `0`--`1` is present in the cycle.
theorem c4_edge01 {
    c4.adj(Nat.0, Nat.1)
} by {
    c4_adj_eq(Nat.0, Nat.1)
    c4.adj(Nat.0, Nat.1) = (c4_part(Nat.0) != c4_part(Nat.1))
    c4_part0
    c4_part(Nat.0) = true
    c4_part1
    c4_part(Nat.1) = false
    c4_part(Nat.0) != c4_part(Nat.1)
    c4.adj(Nat.0, Nat.1)
}

/// The edge `2`--`3` is present in the cycle.
theorem c4_edge23 {
    c4.adj(Nat.2, Nat.3)
} by {
    c4_adj_eq(Nat.2, Nat.3)
    c4.adj(Nat.2, Nat.3) = (c4_part(Nat.2) != c4_part(Nat.3))
    c4_part2
    c4_part(Nat.2) = true
    c4_part3
    c4_part(Nat.3) = false
    c4_part(Nat.2) != c4_part(Nat.3)
    c4.adj(Nat.2, Nat.3)
}

/// The edge `0`--`1` is a directed edge of the cycle.
theorem c4_directed_edge01 {
    directed_edges(c4, range_set(Nat.4)).contains(Pair.new(Nat.0, Nat.1))
} by {
    c4_vertex0
    range_set(Nat.4).contains(Nat.0)
    c4_vertex1
    range_set(Nat.4).contains(Nat.1)
    c4_edge01
    c4.adj(Nat.0, Nat.1)
    directed_edges_contains_pair(c4, range_set(Nat.4), Nat.0, Nat.1)
    directed_edges(c4, range_set(Nat.4)).contains(Pair.new(Nat.0, Nat.1))
}

/// The edge `2`--`3` is a directed edge of the cycle.
theorem c4_directed_edge23 {
    directed_edges(c4, range_set(Nat.4)).contains(Pair.new(Nat.2, Nat.3))
} by {
    c4_vertex2
    range_set(Nat.4).contains(Nat.2)
    c4_vertex3
    range_set(Nat.4).contains(Nat.3)
    c4_edge23
    c4.adj(Nat.2, Nat.3)
    directed_edges_contains_pair(c4, range_set(Nat.4), Nat.2, Nat.3)
    directed_edges(c4, range_set(Nat.4)).contains(Pair.new(Nat.2, Nat.3))
}

/// The two matching edges of the cycle share no vertex.
theorem c4_pairs_not_share {
    not pairs_share_vertex(Pair.new(Nat.0, Nat.1), Pair.new(Nat.2, Nat.3))
} by {
    if Pair.new(Nat.0, Nat.1).first = Pair.new(Nat.2, Nat.3).first {
        Nat.0 = Nat.2
        false
    }
    not (Pair.new(Nat.0, Nat.1).first = Pair.new(Nat.2, Nat.3).first)
    if Pair.new(Nat.0, Nat.1).first = Pair.new(Nat.2, Nat.3).second {
        Nat.0 = Nat.3
        false
    }
    not (Pair.new(Nat.0, Nat.1).first = Pair.new(Nat.2, Nat.3).second)
    if Pair.new(Nat.0, Nat.1).second = Pair.new(Nat.2, Nat.3).first {
        Nat.1 = Nat.2
        false
    }
    not (Pair.new(Nat.0, Nat.1).second = Pair.new(Nat.2, Nat.3).first)
    if Pair.new(Nat.0, Nat.1).second = Pair.new(Nat.2, Nat.3).second {
        Nat.1 = Nat.3
        false
    }
    not (Pair.new(Nat.0, Nat.1).second = Pair.new(Nat.2, Nat.3).second)
    pairs_not_share_of_ne_components(Pair.new(Nat.0, Nat.1), Pair.new(Nat.2, Nat.3))
    not pairs_share_vertex(Pair.new(Nat.0, Nat.1), Pair.new(Nat.2, Nat.3))
}

/// The size-two matching of the cycle: the edges {0, 1} and {2, 3}.
let c4_matching: FiniteSet[Pair[Nat, Nat]] = fs_insert(
    fs_insert(FiniteSet.empty[Pair[Nat, Nat]], Pair.new(Nat.0, Nat.1)), Pair.new(Nat.2, Nat.3))

/// The two-edge set of the cycle is a matching.
theorem c4_matching_is_matching {
    is_matching(c4, range_set(Nat.4), c4_matching)
} by {
    c4_directed_edge01
    directed_edges(c4, range_set(Nat.4)).contains(Pair.new(Nat.0, Nat.1))
    c4_directed_edge23
    directed_edges(c4, range_set(Nat.4)).contains(Pair.new(Nat.2, Nat.3))
    c4_pairs_not_share
    not pairs_share_vertex(Pair.new(Nat.0, Nat.1), Pair.new(Nat.2, Nat.3))
    two_pair_matching_is_matching(c4, range_set(Nat.4), Pair.new(Nat.0, Nat.1), Pair.new(Nat.2, Nat.3))
    is_matching(c4, range_set(Nat.4), fs_insert(fs_insert(FiniteSet.empty[Pair[Nat, Nat]], Pair.new(Nat.0, Nat.1)), Pair.new(Nat.2, Nat.3)))
    c4_matching = fs_insert(fs_insert(FiniteSet.empty[Pair[Nat, Nat]], Pair.new(Nat.0, Nat.1)), Pair.new(Nat.2, Nat.3))
    is_matching(c4, range_set(Nat.4), c4_matching)
}

/// The two-edge matching of the cycle has two pairs.
theorem c4_matching_card {
    fs_card(c4_matching) = Nat.2
} by {
    fs_card_cardinality_is(c4_matching)
    c4_matching.cardinality_is(fs_card(c4_matching))
    fs_card_cardinality_is(fs_insert(FiniteSet.empty[Pair[Nat, Nat]], Pair.new(Nat.0, Nat.1)))
    fs_card_cardinality_is(FiniteSet.empty[Pair[Nat, Nat]])
    fs_card_empty[Pair[Nat, Nat]]
    fs_card(FiniteSet.empty[Pair[Nat, Nat]]) = Nat.0
    FiniteSet.empty[Pair[Nat, Nat]].cardinality_is(Nat.0)
    finite_set_empty_contains_eq[Pair[Nat, Nat]](Pair.new(Nat.0, Nat.1))
    not FiniteSet.empty[Pair[Nat, Nat]].contains(Pair.new(Nat.0, Nat.1))
    finite_set_insert_cardinality_is_suc_of_not_contains(FiniteSet.empty[Pair[Nat, Nat]], Pair.new(Nat.0, Nat.1), Nat.0)
    fs_insert(FiniteSet.empty[Pair[Nat, Nat]], Pair.new(Nat.0, Nat.1)).cardinality_is(Nat.1)
    finite_set_singleton_contains_eq(Pair.new(Nat.0, Nat.1), Pair.new(Nat.2, Nat.3))
    if fs_insert(FiniteSet.empty[Pair[Nat, Nat]], Pair.new(Nat.0, Nat.1)).contains(Pair.new(Nat.2, Nat.3)) {
        fs_insert(FiniteSet.empty[Pair[Nat, Nat]], Pair.new(Nat.0, Nat.1)).contains(Pair.new(Nat.2, Nat.3)) = (Pair.new(Nat.2, Nat.3) = Pair.new(Nat.0, Nat.1))
        Pair.new(Nat.2, Nat.3) = Pair.new(Nat.0, Nat.1)
        Pair.new(Nat.2, Nat.3).first = Pair.new(Nat.0, Nat.1).first
        Pair.new(Nat.2, Nat.3).first = Nat.2
        Pair.new(Nat.0, Nat.1).first = Nat.0
        Nat.2 = Nat.0
        false
    }
    not fs_insert(FiniteSet.empty[Pair[Nat, Nat]], Pair.new(Nat.0, Nat.1)).contains(Pair.new(Nat.2, Nat.3))
    finite_set_insert_cardinality_is_suc_of_not_contains(fs_insert(FiniteSet.empty[Pair[Nat, Nat]], Pair.new(Nat.0, Nat.1)), Pair.new(Nat.2, Nat.3), Nat.1)
    fs_insert(fs_insert(FiniteSet.empty[Pair[Nat, Nat]], Pair.new(Nat.0, Nat.1)), Pair.new(Nat.2, Nat.3)).cardinality_is(Nat.2)
    c4_matching = fs_insert(fs_insert(FiniteSet.empty[Pair[Nat, Nat]], Pair.new(Nat.0, Nat.1)), Pair.new(Nat.2, Nat.3))
    c4_matching.cardinality_is(Nat.2)
    fs_card_eq_of_cardinality_is(c4_matching, Nat.2)
    fs_card(c4_matching) = Nat.2
}

/// No matching of the cycle is larger than two edges.
theorem c4_matching_size_at_most_two(m: FiniteSet[Pair[Nat, Nat]]) {
    is_matching(c4, range_set(Nat.4), m) implies fs_card(m) <= Nat.2
} by {
    if is_matching(c4, range_set(Nat.4), m) {
        matching_size_le_half_vertices(c4, range_set(Nat.4), m)
        fs_card(m) + fs_card(m) <= fs_card(range_set(Nat.4))
        range_set_card(Nat.4)
        fs_card(range_set(Nat.4)) = Nat.4
        fs_card(m) + fs_card(m) <= Nat.4
        double_lte_four(fs_card(m))
        fs_card(m) <= Nat.2
    }
}

/// Two is an achieved matching size of the cycle.
theorem c4_matching_size_pred_two {
    matching_size_pred(c4, range_set(Nat.4))(Nat.2)
} by {
    c4_matching_is_matching
    is_matching(c4, range_set(Nat.4), c4_matching)
    c4_matching_card
    fs_card(c4_matching) = Nat.2
    matching_size_pred_intro(c4, range_set(Nat.4), c4_matching)
    matching_size_pred(c4, range_set(Nat.4))(fs_card(c4_matching))
    matching_size_pred(c4, range_set(Nat.4))(Nat.2)
}

/// Every achieved matching size of the cycle is at most two.
theorem c4_matching_size_pred_bounded(n: Nat) {
    matching_size_pred(c4, range_set(Nat.4))(n) implies n <= Nat.2
} by {
    if matching_size_pred(c4, range_set(Nat.4))(n) {
        matching_size_pred(c4, range_set(Nat.4))(n) = exists(m: FiniteSet[Pair[Nat, Nat]]) {
            is_matching(c4, range_set(Nat.4), m) and fs_card(m) = n
        }
        let m: FiniteSet[Pair[Nat, Nat]] satisfy {
            is_matching(c4, range_set(Nat.4), m) and fs_card(m) = n
        }
        c4_matching_size_at_most_two(m)
        fs_card(m) <= Nat.2
        fs_card(m) = n
        n <= Nat.2
    }
}

/// The maximum matching size of the cycle is two.
theorem c4_maximum_matching_size {
    maximum_matching_size(c4, range_set(Nat.4)) = Nat.2
} by {
    c4_matching_size_pred_two
    matching_size_pred(c4, range_set(Nat.4))(Nat.2)
    forall(n: Nat) {
        c4_matching_size_pred_bounded(n)
        matching_size_pred(c4, range_set(Nat.4))(n) implies n <= Nat.2
    }
    is_upper_bound_of_intro(matching_size_pred(c4, range_set(Nat.4)), Nat.2)
    is_upper_bound_of(matching_size_pred(c4, range_set(Nat.4)), Nat.2)
    is_max_intro(matching_size_pred(c4, range_set(Nat.4)), Nat.2)
    is_max(matching_size_pred(c4, range_set(Nat.4)), Nat.2)
    is_max(matching_size_pred(c4, range_set(Nat.4)), maximum_matching_size(c4, range_set(Nat.4)))
    max_unique(matching_size_pred(c4, range_set(Nat.4)), Nat.2, maximum_matching_size(c4, range_set(Nat.4)))
    maximum_matching_size(c4, range_set(Nat.4)) = Nat.2
}

/// The size-two vertex cover of the cycle: one vertex from each opposite pair.
let c4_cover: FiniteSet[Nat] = fs_insert(fs_insert(FiniteSet.empty[Nat], Nat.0), Nat.2)

/// An edge of the cycle meets the cover {0, 2}.
theorem c4_adj_covered(x: Nat, y: Nat) {
    c4.adj(x, y) implies c4_cover.contains(x) or c4_cover.contains(y)
} by {
    if c4.adj(x, y) {
        c4_adj_eq(x, y)
        c4_part(x) != c4_part(y)
        c4_part(x) = (x = Nat.0 or x = Nat.2)
        c4_part(y) = (y = Nat.0 or y = Nat.2)
        finite_set_singleton_contains_eq(Nat.0, x)
        finite_set_singleton_contains_eq(Nat.2, x)
        finite_set_singleton_contains_eq(Nat.0, y)
        finite_set_singleton_contains_eq(Nat.2, y)
        if c4_part(x) {
            if x = Nat.0 {
                fs_insert(FiniteSet.empty[Nat], Nat.0).contains(x)
                c4_cover.contains(x)
                c4_cover.contains(x) or c4_cover.contains(y)
            }
            if not (x = Nat.0) {
                x = Nat.2
                c4_cover.contains(x)
                c4_cover.contains(x) or c4_cover.contains(y)
            }
            c4_cover.contains(x) or c4_cover.contains(y)
        }
        if not c4_part(x) {
            c4_part(y)
            if y = Nat.0 {
                fs_insert(FiniteSet.empty[Nat], Nat.0).contains(y)
                c4_cover.contains(y)
                c4_cover.contains(x) or c4_cover.contains(y)
            }
            if not (y = Nat.0) {
                y = Nat.2
                c4_cover.contains(y)
                c4_cover.contains(x) or c4_cover.contains(y)
            }
            c4_cover.contains(x) or c4_cover.contains(y)
        }
        c4_cover.contains(x) or c4_cover.contains(y)
    }
}

/// The set {0, 2} covers every edge of the cycle.
theorem c4_cover_is_vertex_cover {
    is_vertex_cover(c4, range_set(Nat.4), c4_cover)
} by {
    forall(x: Nat, y: Nat) {
        if range_set(Nat.4).contains(x) and range_set(Nat.4).contains(y) and c4.adj(x, y) {
            c4_adj_covered(x, y)
            c4_cover.contains(x) or c4_cover.contains(y)
        }
        (range_set(Nat.4).contains(x) and range_set(Nat.4).contains(y) and c4.adj(x, y)) implies c4_cover.contains(x) or c4_cover.contains(y)
    }
    is_vertex_cover_intro(c4, range_set(Nat.4), c4_cover)
    is_vertex_cover(c4, range_set(Nat.4), c4_cover)
}

/// The cover {0, 2} lies in the vertex set of the cycle.
theorem c4_cover_subset {
    c4_cover.subset_eq(range_set(Nat.4))
} by {
    forall(x: Nat) {
        if c4_cover.contains(x) {
            fs_insert_contains_eq(fs_insert(FiniteSet.empty[Nat], Nat.0), Nat.2, x)
            finite_set_singleton_contains_eq(Nat.0, x)
            c4_cover.contains(x) = (x = Nat.2 or (x = Nat.0))
            if x = Nat.0 {
                c4_vertices_lt
                Nat.0 < Nat.4
                range_set_contains(Nat.4, Nat.0)
                range_set(Nat.4).contains(Nat.0)
                range_set(Nat.4).contains(x)
            }
            if not (x = Nat.0) {
                x = Nat.2
                c4_vertices_lt
                Nat.2 < Nat.4
                range_set_contains(Nat.4, Nat.2)
                range_set(Nat.4).contains(Nat.2)
                range_set(Nat.4).contains(x)
            }
            range_set(Nat.4).contains(x)
        }
        c4_cover.contains(x) implies range_set(Nat.4).contains(x)
    }
    fs_subset_eq_intro(c4_cover, range_set(Nat.4))
    c4_cover.subset_eq(range_set(Nat.4))
}

/// The cover {0, 2} has two vertices.
theorem c4_cover_card {
    fs_card(c4_cover) = Nat.2
} by {
    fs_card_cardinality_is(c4_cover)
    c4_cover.cardinality_is(fs_card(c4_cover))
    fs_card_cardinality_is(fs_insert(FiniteSet.empty[Nat], Nat.0))
    fs_card_cardinality_is(FiniteSet.empty[Nat])
    fs_card_empty[Nat]
    fs_card(FiniteSet.empty[Nat]) = Nat.0
    FiniteSet.empty[Nat].cardinality_is(Nat.0)
    finite_set_empty_contains_eq[Nat](Nat.0)
    not FiniteSet.empty[Nat].contains(Nat.0)
    finite_set_insert_cardinality_is_suc_of_not_contains(FiniteSet.empty[Nat], Nat.0, Nat.0)
    fs_insert(FiniteSet.empty[Nat], Nat.0).cardinality_is(Nat.1)
    finite_set_singleton_contains_eq(Nat.0, Nat.2)
    not fs_insert(FiniteSet.empty[Nat], Nat.0).contains(Nat.2)
    finite_set_insert_cardinality_is_suc_of_not_contains(fs_insert(FiniteSet.empty[Nat], Nat.0), Nat.2, Nat.1)
    fs_insert(fs_insert(FiniteSet.empty[Nat], Nat.0), Nat.2).cardinality_is(Nat.2)
    c4_cover = fs_insert(fs_insert(FiniteSet.empty[Nat], Nat.0), Nat.2)
    c4_cover.cardinality_is(Nat.2)
    fs_card_eq_of_cardinality_is(c4_cover, Nat.2)
    fs_card(c4_cover) = Nat.2
}

/// Two is an achieved cover size of the cycle.
theorem c4_cover_size_pred_two {
    cover_size_pred(c4, range_set(Nat.4))(Nat.2)
} by {
    c4_cover_subset
    c4_cover.subset_eq(range_set(Nat.4))
    c4_cover_is_vertex_cover
    is_vertex_cover(c4, range_set(Nat.4), c4_cover)
    c4_cover_card
    fs_card(c4_cover) = Nat.2
    cover_size_pred_intro(c4, range_set(Nat.4), c4_cover)
    cover_size_pred(c4, range_set(Nat.4))(fs_card(c4_cover))
    cover_size_pred(c4, range_set(Nat.4))(Nat.2)
}

/// Every achieved cover size of the cycle is at least two.
theorem c4_cover_size_pred_bounded(n: Nat) {
    cover_size_pred(c4, range_set(Nat.4))(n) implies Nat.2 <= n
} by {
    if cover_size_pred(c4, range_set(Nat.4))(n) {
        cover_size_pred(c4, range_set(Nat.4))(n) = exists(c: FiniteSet[Nat]) {
            c.subset_eq(range_set(Nat.4)) and is_vertex_cover(c4, range_set(Nat.4), c) and fs_card(c) = n
        }
        let c: FiniteSet[Nat] satisfy {
            c.subset_eq(range_set(Nat.4)) and is_vertex_cover(c4, range_set(Nat.4), c) and fs_card(c) = n
        }
        c4_matching_is_matching
        is_matching(c4, range_set(Nat.4), c4_matching)
        vertex_cover_bounds_matching(c4, range_set(Nat.4), c, c4_matching)
        fs_card(c4_matching) <= fs_card(c)
        c4_matching_card
        fs_card(c4_matching) = Nat.2
        Nat.2 <= fs_card(c)
        fs_card(c) = n
        Nat.2 <= n
    }
}

/// The minimum vertex cover number of the cycle is two.
theorem c4_vertex_cover_number {
    vertex_cover_number(c4, range_set(Nat.4)) = Nat.2
} by {
    c4_cover_size_pred_two
    cover_size_pred(c4, range_set(Nat.4))(Nat.2)
    forall(x: Nat) {
        if x < Nat.2 {
            if cover_size_pred(c4, range_set(Nat.4))(x) {
                c4_cover_size_pred_bounded(x)
                Nat.2 <= x
                lte_and_lt(Nat.2, x, Nat.2)
                Nat.2 < Nat.2
                lt_not_ref(Nat.2)
                false
            }
            not cover_size_pred(c4, range_set(Nat.4))(x)
        }
        x < Nat.2 implies not cover_size_pred(c4, range_set(Nat.4))(x)
    }
    false_below(cover_size_pred(c4, range_set(Nat.4)), Nat.2)
    is_min(cover_size_pred(c4, range_set(Nat.4)), Nat.2)
    is_min(cover_size_pred(c4, range_set(Nat.4)), vertex_cover_number(c4, range_set(Nat.4)))
    min_unique(cover_size_pred(c4, range_set(Nat.4)), Nat.2, vertex_cover_number(c4, range_set(Nat.4)))
    vertex_cover_number(c4, range_set(Nat.4)) = Nat.2
}

/// König's equality on the cycle C4.
theorem konig_on_c4 {
    is_bipartite(c4, range_set(Nat.4)) and
        maximum_matching_size(c4, range_set(Nat.4)) = vertex_cover_number(c4, range_set(Nat.4))
} by {
    c4_is_bipartite
    is_bipartite(c4, range_set(Nat.4))
    c4_maximum_matching_size
    maximum_matching_size(c4, range_set(Nat.4)) = Nat.2
    c4_vertex_cover_number
    vertex_cover_number(c4, range_set(Nat.4)) = Nat.2
    maximum_matching_size(c4, range_set(Nat.4)) = vertex_cover_number(c4, range_set(Nat.4))
}

// --- The complete bipartite graph K_{2,2}: vertices 0, 1 on one side and 2, 3 on the other. ---

/// The two-colouring of K_{2,2}: the first two vertices form one side.
define k22_part(v: Nat) -> Bool {
    v = Nat.0 or v = Nat.1
}

/// Adjacency of K_{2,2}: the two endpoints lie on opposite sides.
define k22_adj(x: Nat, y: Nat) -> Bool {
    k22_part(x) != k22_part(y)
}

/// The K_{2,2} adjacency relation is symmetric.
theorem k22_adj_is_symmetric {
    is_symmetric(k22_adj)
} by {
    forall(x: Nat, y: Nat) {
        if k22_adj(x, y) {
            k22_adj(x, y) = (k22_part(x) != k22_part(y))
            k22_part(x) != k22_part(y)
            k22_part(y) != k22_part(x)
            k22_adj(y, x) = (k22_part(y) != k22_part(x))
            k22_adj(y, x)
        }
        k22_adj(x, y) implies k22_adj(y, x)
    }
}

/// The K_{2,2} adjacency relation has no loops.
theorem k22_adj_is_irreflexive {
    is_irreflexive(k22_adj)
} by {
    forall(x: Nat) {
        if k22_adj(x, x) {
            k22_adj(x, x) = (k22_part(x) != k22_part(x))
            false
        }
        not k22_adj(x, x)
    }
}

/// The K_{2,2} adjacency relation satisfies the simple-graph constraint.
theorem k22_adj_constraint {
    is_symmetric(k22_adj) and is_irreflexive(k22_adj)
} by {
    k22_adj_is_symmetric
    k22_adj_is_irreflexive
}

/// The complete bipartite graph K_{2,2} exists.
theorem k22_graph_exists {
    exists(g: SimpleGraph[Nat]) { SimpleGraph.new(k22_adj) = Option.some(g) }
} by {
    k22_adj_constraint
}

/// The complete bipartite graph K_{2,2}.
let k22: SimpleGraph[Nat] satisfy {
    SimpleGraph.new(k22_adj) = Option.some(k22)
}

/// Adjacency in K_{2,2} is the K_{2,2} adjacency relation.
theorem k22_adj_eq(x: Nat, y: Nat) {
    k22.adj(x, y) = (k22_part(x) != k22_part(y))
} by {
    k22.adj = k22_adj
    k22.adj(x, y) = k22_adj(x, y)
    k22_adj(x, y) = (k22_part(x) != k22_part(y))
}

/// The graph K_{2,2} is bipartite.
theorem k22_is_bipartition {
    is_bipartition(k22, range_set(Nat.4), k22_part)
} by {
    forall(x: Nat, y: Nat) {
        if range_set(Nat.4).contains(x) and range_set(Nat.4).contains(y) and k22.adj(x, y) {
            k22_adj_eq(x, y)
            k22_part(x) != k22_part(y)
        }
        (range_set(Nat.4).contains(x) and range_set(Nat.4).contains(y) and k22.adj(x, y)) implies k22_part(x) != k22_part(y)
    }
    is_bipartition_intro(k22, range_set(Nat.4), k22_part)
    is_bipartition(k22, range_set(Nat.4), k22_part)
}

/// The graph K_{2,2} is bipartite.
theorem k22_is_bipartite {
    is_bipartite(k22, range_set(Nat.4))
} by {
    k22_is_bipartition
    is_bipartite_intro(k22, range_set(Nat.4), k22_part)
    is_bipartite(k22, range_set(Nat.4))
}

/// The colour of `0` is true.
theorem k22_part0 {
    k22_part(Nat.0) = true
} by {
    k22_part(Nat.0) = (Nat.0 = Nat.0 or Nat.0 = Nat.1)
    Nat.0 = Nat.0
    k22_part(Nat.0) = true
}

/// The colour of `1` is true.
theorem k22_part1 {
    k22_part(Nat.1) = true
} by {
    k22_part(Nat.1) = (Nat.1 = Nat.0 or Nat.1 = Nat.1)
    Nat.1 = Nat.1
    k22_part(Nat.1) = true
}

/// The colour of `2` is false.
theorem k22_part2 {
    k22_part(Nat.2) = false
} by {
    k22_part(Nat.2) = (Nat.2 = Nat.0 or Nat.2 = Nat.1)
    Nat.2 != Nat.0
    Nat.2 != Nat.1
    part_false_of_ne(Nat.2, Nat.0, Nat.1)
    (Nat.2 = Nat.0 or Nat.2 = Nat.1) = false
    k22_part(Nat.2) = false
}

/// The colour of `3` is false.
theorem k22_part3 {
    k22_part(Nat.3) = false
} by {
    k22_part(Nat.3) = (Nat.3 = Nat.0 or Nat.3 = Nat.1)
    Nat.3 != Nat.0
    Nat.3 != Nat.1
    part_false_of_ne(Nat.3, Nat.0, Nat.1)
    (Nat.3 = Nat.0 or Nat.3 = Nat.1) = false
    k22_part(Nat.3) = false
}

/// The edge `0`--`2` is present in K_{2,2}.
theorem k22_edge02 {
    k22.adj(Nat.0, Nat.2)
} by {
    k22_adj_eq(Nat.0, Nat.2)
    k22.adj(Nat.0, Nat.2) = (k22_part(Nat.0) != k22_part(Nat.2))
    k22_part0
    k22_part(Nat.0) = true
    k22_part2
    k22_part(Nat.2) = false
    k22_part(Nat.0) != k22_part(Nat.2)
    k22.adj(Nat.0, Nat.2)
}

/// The edge `1`--`3` is present in K_{2,2}.
theorem k22_edge13 {
    k22.adj(Nat.1, Nat.3)
} by {
    k22_adj_eq(Nat.1, Nat.3)
    k22.adj(Nat.1, Nat.3) = (k22_part(Nat.1) != k22_part(Nat.3))
    k22_part1
    k22_part(Nat.1) = true
    k22_part3
    k22_part(Nat.3) = false
    k22_part(Nat.1) != k22_part(Nat.3)
    k22.adj(Nat.1, Nat.3)
}

/// The edge `0`--`2` is a directed edge of K_{2,2}.
theorem k22_directed_edge02 {
    directed_edges(k22, range_set(Nat.4)).contains(Pair.new(Nat.0, Nat.2))
} by {
    c4_vertex0
    range_set(Nat.4).contains(Nat.0)
    c4_vertex2
    range_set(Nat.4).contains(Nat.2)
    k22_edge02
    k22.adj(Nat.0, Nat.2)
    directed_edges_contains_pair(k22, range_set(Nat.4), Nat.0, Nat.2)
    directed_edges(k22, range_set(Nat.4)).contains(Pair.new(Nat.0, Nat.2))
}

/// The edge `1`--`3` is a directed edge of K_{2,2}.
theorem k22_directed_edge13 {
    directed_edges(k22, range_set(Nat.4)).contains(Pair.new(Nat.1, Nat.3))
} by {
    c4_vertex1
    range_set(Nat.4).contains(Nat.1)
    c4_vertex3
    range_set(Nat.4).contains(Nat.3)
    k22_edge13
    k22.adj(Nat.1, Nat.3)
    directed_edges_contains_pair(k22, range_set(Nat.4), Nat.1, Nat.3)
    directed_edges(k22, range_set(Nat.4)).contains(Pair.new(Nat.1, Nat.3))
}

/// The two matching edges of K_{2,2} share no vertex.
theorem k22_pairs_not_share {
    not pairs_share_vertex(Pair.new(Nat.0, Nat.2), Pair.new(Nat.1, Nat.3))
} by {
    if Pair.new(Nat.0, Nat.2).first = Pair.new(Nat.1, Nat.3).first {
        Nat.0 = Nat.1
        false
    }
    not (Pair.new(Nat.0, Nat.2).first = Pair.new(Nat.1, Nat.3).first)
    if Pair.new(Nat.0, Nat.2).first = Pair.new(Nat.1, Nat.3).second {
        Nat.0 = Nat.3
        false
    }
    not (Pair.new(Nat.0, Nat.2).first = Pair.new(Nat.1, Nat.3).second)
    if Pair.new(Nat.0, Nat.2).second = Pair.new(Nat.1, Nat.3).first {
        Nat.2 = Nat.1
        false
    }
    not (Pair.new(Nat.0, Nat.2).second = Pair.new(Nat.1, Nat.3).first)
    if Pair.new(Nat.0, Nat.2).second = Pair.new(Nat.1, Nat.3).second {
        Nat.2 = Nat.3
        false
    }
    not (Pair.new(Nat.0, Nat.2).second = Pair.new(Nat.1, Nat.3).second)
    pairs_not_share_of_ne_components(Pair.new(Nat.0, Nat.2), Pair.new(Nat.1, Nat.3))
    not pairs_share_vertex(Pair.new(Nat.0, Nat.2), Pair.new(Nat.1, Nat.3))
}

/// The size-two matching of K_{2,2}: the edges {0, 2} and {1, 3}.
let k22_matching: FiniteSet[Pair[Nat, Nat]] = fs_insert(
    fs_insert(FiniteSet.empty[Pair[Nat, Nat]], Pair.new(Nat.0, Nat.2)), Pair.new(Nat.1, Nat.3))

/// The two-edge set of K_{2,2} is a matching.
theorem k22_matching_is_matching {
    is_matching(k22, range_set(Nat.4), k22_matching)
} by {
    k22_directed_edge02
    directed_edges(k22, range_set(Nat.4)).contains(Pair.new(Nat.0, Nat.2))
    k22_directed_edge13
    directed_edges(k22, range_set(Nat.4)).contains(Pair.new(Nat.1, Nat.3))
    k22_pairs_not_share
    not pairs_share_vertex(Pair.new(Nat.0, Nat.2), Pair.new(Nat.1, Nat.3))
    two_pair_matching_is_matching(k22, range_set(Nat.4), Pair.new(Nat.0, Nat.2), Pair.new(Nat.1, Nat.3))
    is_matching(k22, range_set(Nat.4), fs_insert(fs_insert(FiniteSet.empty[Pair[Nat, Nat]], Pair.new(Nat.0, Nat.2)), Pair.new(Nat.1, Nat.3)))
    k22_matching = fs_insert(fs_insert(FiniteSet.empty[Pair[Nat, Nat]], Pair.new(Nat.0, Nat.2)), Pair.new(Nat.1, Nat.3))
    is_matching(k22, range_set(Nat.4), k22_matching)
}

/// The two-edge matching of K_{2,2} has two pairs.
theorem k22_matching_card {
    fs_card(k22_matching) = Nat.2
} by {
    fs_card_cardinality_is(k22_matching)
    k22_matching.cardinality_is(fs_card(k22_matching))
    fs_card_cardinality_is(fs_insert(FiniteSet.empty[Pair[Nat, Nat]], Pair.new(Nat.0, Nat.2)))
    fs_card_cardinality_is(FiniteSet.empty[Pair[Nat, Nat]])
    fs_card_empty[Pair[Nat, Nat]]
    fs_card(FiniteSet.empty[Pair[Nat, Nat]]) = Nat.0
    FiniteSet.empty[Pair[Nat, Nat]].cardinality_is(Nat.0)
    finite_set_empty_contains_eq[Pair[Nat, Nat]](Pair.new(Nat.0, Nat.2))
    not FiniteSet.empty[Pair[Nat, Nat]].contains(Pair.new(Nat.0, Nat.2))
    finite_set_insert_cardinality_is_suc_of_not_contains(FiniteSet.empty[Pair[Nat, Nat]], Pair.new(Nat.0, Nat.2), Nat.0)
    fs_insert(FiniteSet.empty[Pair[Nat, Nat]], Pair.new(Nat.0, Nat.2)).cardinality_is(Nat.1)
    finite_set_singleton_contains_eq(Pair.new(Nat.0, Nat.2), Pair.new(Nat.1, Nat.3))
    if fs_insert(FiniteSet.empty[Pair[Nat, Nat]], Pair.new(Nat.0, Nat.2)).contains(Pair.new(Nat.1, Nat.3)) {
        fs_insert(FiniteSet.empty[Pair[Nat, Nat]], Pair.new(Nat.0, Nat.2)).contains(Pair.new(Nat.1, Nat.3)) = (Pair.new(Nat.1, Nat.3) = Pair.new(Nat.0, Nat.2))
        Pair.new(Nat.1, Nat.3) = Pair.new(Nat.0, Nat.2)
        Pair.new(Nat.1, Nat.3).first = Pair.new(Nat.0, Nat.2).first
        Pair.new(Nat.1, Nat.3).first = Nat.1
        Pair.new(Nat.0, Nat.2).first = Nat.0
        Nat.1 = Nat.0
        false
    }
    not fs_insert(FiniteSet.empty[Pair[Nat, Nat]], Pair.new(Nat.0, Nat.2)).contains(Pair.new(Nat.1, Nat.3))
    finite_set_insert_cardinality_is_suc_of_not_contains(fs_insert(FiniteSet.empty[Pair[Nat, Nat]], Pair.new(Nat.0, Nat.2)), Pair.new(Nat.1, Nat.3), Nat.1)
    fs_insert(fs_insert(FiniteSet.empty[Pair[Nat, Nat]], Pair.new(Nat.0, Nat.2)), Pair.new(Nat.1, Nat.3)).cardinality_is(Nat.2)
    k22_matching = fs_insert(fs_insert(FiniteSet.empty[Pair[Nat, Nat]], Pair.new(Nat.0, Nat.2)), Pair.new(Nat.1, Nat.3))
    k22_matching.cardinality_is(Nat.2)
    fs_card_eq_of_cardinality_is(k22_matching, Nat.2)
    fs_card(k22_matching) = Nat.2
}

/// No matching of K_{2,2} is larger than two edges.
theorem k22_matching_size_at_most_two(m: FiniteSet[Pair[Nat, Nat]]) {
    is_matching(k22, range_set(Nat.4), m) implies fs_card(m) <= Nat.2
} by {
    if is_matching(k22, range_set(Nat.4), m) {
        matching_size_le_half_vertices(k22, range_set(Nat.4), m)
        fs_card(m) + fs_card(m) <= fs_card(range_set(Nat.4))
        range_set_card(Nat.4)
        fs_card(range_set(Nat.4)) = Nat.4
        fs_card(m) + fs_card(m) <= Nat.4
        double_lte_four(fs_card(m))
        fs_card(m) <= Nat.2
    }
}

/// Two is an achieved matching size of K_{2,2}.
theorem k22_matching_size_pred_two {
    matching_size_pred(k22, range_set(Nat.4))(Nat.2)
} by {
    k22_matching_is_matching
    is_matching(k22, range_set(Nat.4), k22_matching)
    k22_matching_card
    fs_card(k22_matching) = Nat.2
    matching_size_pred_intro(k22, range_set(Nat.4), k22_matching)
    matching_size_pred(k22, range_set(Nat.4))(fs_card(k22_matching))
    matching_size_pred(k22, range_set(Nat.4))(Nat.2)
}

/// Every achieved matching size of K_{2,2} is at most two.
theorem k22_matching_size_pred_bounded(n: Nat) {
    matching_size_pred(k22, range_set(Nat.4))(n) implies n <= Nat.2
} by {
    if matching_size_pred(k22, range_set(Nat.4))(n) {
        matching_size_pred(k22, range_set(Nat.4))(n) = exists(m: FiniteSet[Pair[Nat, Nat]]) {
            is_matching(k22, range_set(Nat.4), m) and fs_card(m) = n
        }
        let m: FiniteSet[Pair[Nat, Nat]] satisfy {
            is_matching(k22, range_set(Nat.4), m) and fs_card(m) = n
        }
        k22_matching_size_at_most_two(m)
        fs_card(m) <= Nat.2
        fs_card(m) = n
        n <= Nat.2
    }
}

/// The maximum matching size of K_{2,2} is two.
theorem k22_maximum_matching_size {
    maximum_matching_size(k22, range_set(Nat.4)) = Nat.2
} by {
    k22_matching_size_pred_two
    matching_size_pred(k22, range_set(Nat.4))(Nat.2)
    forall(n: Nat) {
        k22_matching_size_pred_bounded(n)
        matching_size_pred(k22, range_set(Nat.4))(n) implies n <= Nat.2
    }
    is_upper_bound_of_intro(matching_size_pred(k22, range_set(Nat.4)), Nat.2)
    is_upper_bound_of(matching_size_pred(k22, range_set(Nat.4)), Nat.2)
    is_max_intro(matching_size_pred(k22, range_set(Nat.4)), Nat.2)
    is_max(matching_size_pred(k22, range_set(Nat.4)), Nat.2)
    is_max(matching_size_pred(k22, range_set(Nat.4)), maximum_matching_size(k22, range_set(Nat.4)))
    max_unique(matching_size_pred(k22, range_set(Nat.4)), Nat.2, maximum_matching_size(k22, range_set(Nat.4)))
    maximum_matching_size(k22, range_set(Nat.4)) = Nat.2
}

/// The size-two vertex cover of K_{2,2}: one whole side.
let k22_cover: FiniteSet[Nat] = fs_insert(fs_insert(FiniteSet.empty[Nat], Nat.0), Nat.1)

/// An edge of K_{2,2} meets the cover {0, 1}.
theorem k22_adj_covered(x: Nat, y: Nat) {
    k22.adj(x, y) implies k22_cover.contains(x) or k22_cover.contains(y)
} by {
    if k22.adj(x, y) {
        k22_adj_eq(x, y)
        k22_part(x) != k22_part(y)
        k22_part(x) = (x = Nat.0 or x = Nat.1)
        k22_part(y) = (y = Nat.0 or y = Nat.1)
        finite_set_singleton_contains_eq(Nat.0, x)
        finite_set_singleton_contains_eq(Nat.1, x)
        finite_set_singleton_contains_eq(Nat.0, y)
        finite_set_singleton_contains_eq(Nat.1, y)
        if k22_part(x) {
            if x = Nat.0 {
                fs_insert(FiniteSet.empty[Nat], Nat.0).contains(x)
                k22_cover.contains(x)
                k22_cover.contains(x) or k22_cover.contains(y)
            }
            if not (x = Nat.0) {
                x = Nat.1
                k22_cover.contains(x)
                k22_cover.contains(x) or k22_cover.contains(y)
            }
            k22_cover.contains(x) or k22_cover.contains(y)
        }
        if not k22_part(x) {
            k22_part(y)
            if y = Nat.0 {
                fs_insert(FiniteSet.empty[Nat], Nat.0).contains(y)
                k22_cover.contains(y)
                k22_cover.contains(x) or k22_cover.contains(y)
            }
            if not (y = Nat.0) {
                y = Nat.1
                k22_cover.contains(y)
                k22_cover.contains(x) or k22_cover.contains(y)
            }
            k22_cover.contains(x) or k22_cover.contains(y)
        }
        k22_cover.contains(x) or k22_cover.contains(y)
    }
}

/// The set {0, 1} covers every edge of K_{2,2}.
theorem k22_cover_is_vertex_cover {
    is_vertex_cover(k22, range_set(Nat.4), k22_cover)
} by {
    forall(x: Nat, y: Nat) {
        if range_set(Nat.4).contains(x) and range_set(Nat.4).contains(y) and k22.adj(x, y) {
            k22_adj_covered(x, y)
            k22_cover.contains(x) or k22_cover.contains(y)
        }
        (range_set(Nat.4).contains(x) and range_set(Nat.4).contains(y) and k22.adj(x, y)) implies k22_cover.contains(x) or k22_cover.contains(y)
    }
    is_vertex_cover_intro(k22, range_set(Nat.4), k22_cover)
    is_vertex_cover(k22, range_set(Nat.4), k22_cover)
}

/// The cover {0, 1} lies in the vertex set of K_{2,2}.
theorem k22_cover_subset {
    k22_cover.subset_eq(range_set(Nat.4))
} by {
    forall(x: Nat) {
        if k22_cover.contains(x) {
            fs_insert_contains_eq(fs_insert(FiniteSet.empty[Nat], Nat.0), Nat.1, x)
            finite_set_singleton_contains_eq(Nat.0, x)
            k22_cover.contains(x) = (x = Nat.1 or (x = Nat.0))
            if x = Nat.0 {
                c4_vertices_lt
                Nat.0 < Nat.4
                range_set_contains(Nat.4, Nat.0)
                range_set(Nat.4).contains(Nat.0)
                range_set(Nat.4).contains(x)
            }
            if not (x = Nat.0) {
                x = Nat.1
                c4_vertices_lt
                Nat.1 < Nat.4
                range_set_contains(Nat.4, Nat.1)
                range_set(Nat.4).contains(Nat.1)
                range_set(Nat.4).contains(x)
            }
            range_set(Nat.4).contains(x)
        }
        k22_cover.contains(x) implies range_set(Nat.4).contains(x)
    }
    fs_subset_eq_intro(k22_cover, range_set(Nat.4))
    k22_cover.subset_eq(range_set(Nat.4))
}

/// The cover {0, 1} has two vertices.
theorem k22_cover_card {
    fs_card(k22_cover) = Nat.2
} by {
    fs_card_cardinality_is(k22_cover)
    k22_cover.cardinality_is(fs_card(k22_cover))
    fs_card_cardinality_is(fs_insert(FiniteSet.empty[Nat], Nat.0))
    fs_card_cardinality_is(FiniteSet.empty[Nat])
    fs_card_empty[Nat]
    fs_card(FiniteSet.empty[Nat]) = Nat.0
    FiniteSet.empty[Nat].cardinality_is(Nat.0)
    finite_set_empty_contains_eq[Nat](Nat.0)
    not FiniteSet.empty[Nat].contains(Nat.0)
    finite_set_insert_cardinality_is_suc_of_not_contains(FiniteSet.empty[Nat], Nat.0, Nat.0)
    fs_insert(FiniteSet.empty[Nat], Nat.0).cardinality_is(Nat.1)
    finite_set_singleton_contains_eq(Nat.0, Nat.1)
    not fs_insert(FiniteSet.empty[Nat], Nat.0).contains(Nat.1)
    finite_set_insert_cardinality_is_suc_of_not_contains(fs_insert(FiniteSet.empty[Nat], Nat.0), Nat.1, Nat.1)
    fs_insert(fs_insert(FiniteSet.empty[Nat], Nat.0), Nat.1).cardinality_is(Nat.2)
    k22_cover = fs_insert(fs_insert(FiniteSet.empty[Nat], Nat.0), Nat.1)
    k22_cover.cardinality_is(Nat.2)
    fs_card_eq_of_cardinality_is(k22_cover, Nat.2)
    fs_card(k22_cover) = Nat.2
}

/// Two is an achieved cover size of K_{2,2}.
theorem k22_cover_size_pred_two {
    cover_size_pred(k22, range_set(Nat.4))(Nat.2)
} by {
    k22_cover_subset
    k22_cover.subset_eq(range_set(Nat.4))
    k22_cover_is_vertex_cover
    is_vertex_cover(k22, range_set(Nat.4), k22_cover)
    k22_cover_card
    fs_card(k22_cover) = Nat.2
    cover_size_pred_intro(k22, range_set(Nat.4), k22_cover)
    cover_size_pred(k22, range_set(Nat.4))(fs_card(k22_cover))
    cover_size_pred(k22, range_set(Nat.4))(Nat.2)
}

/// Every achieved cover size of K_{2,2} is at least two.
theorem k22_cover_size_pred_bounded(n: Nat) {
    cover_size_pred(k22, range_set(Nat.4))(n) implies Nat.2 <= n
} by {
    if cover_size_pred(k22, range_set(Nat.4))(n) {
        cover_size_pred(k22, range_set(Nat.4))(n) = exists(c: FiniteSet[Nat]) {
            c.subset_eq(range_set(Nat.4)) and is_vertex_cover(k22, range_set(Nat.4), c) and fs_card(c) = n
        }
        let c: FiniteSet[Nat] satisfy {
            c.subset_eq(range_set(Nat.4)) and is_vertex_cover(k22, range_set(Nat.4), c) and fs_card(c) = n
        }
        k22_matching_is_matching
        is_matching(k22, range_set(Nat.4), k22_matching)
        vertex_cover_bounds_matching(k22, range_set(Nat.4), c, k22_matching)
        fs_card(k22_matching) <= fs_card(c)
        k22_matching_card
        fs_card(k22_matching) = Nat.2
        Nat.2 <= fs_card(c)
        fs_card(c) = n
        Nat.2 <= n
    }
}

/// The minimum vertex cover number of K_{2,2} is two.
theorem k22_vertex_cover_number {
    vertex_cover_number(k22, range_set(Nat.4)) = Nat.2
} by {
    k22_cover_size_pred_two
    cover_size_pred(k22, range_set(Nat.4))(Nat.2)
    forall(x: Nat) {
        if x < Nat.2 {
            if cover_size_pred(k22, range_set(Nat.4))(x) {
                k22_cover_size_pred_bounded(x)
                Nat.2 <= x
                lte_and_lt(Nat.2, x, Nat.2)
                Nat.2 < Nat.2
                lt_not_ref(Nat.2)
                false
            }
            not cover_size_pred(k22, range_set(Nat.4))(x)
        }
        x < Nat.2 implies not cover_size_pred(k22, range_set(Nat.4))(x)
    }
    false_below(cover_size_pred(k22, range_set(Nat.4)), Nat.2)
    is_min(cover_size_pred(k22, range_set(Nat.4)), Nat.2)
    is_min(cover_size_pred(k22, range_set(Nat.4)), vertex_cover_number(k22, range_set(Nat.4)))
    min_unique(cover_size_pred(k22, range_set(Nat.4)), Nat.2, vertex_cover_number(k22, range_set(Nat.4)))
    vertex_cover_number(k22, range_set(Nat.4)) = Nat.2
}

/// König's equality on the complete bipartite graph K_{2,2}.
theorem konig_on_k22 {
    is_bipartite(k22, range_set(Nat.4)) and
        maximum_matching_size(k22, range_set(Nat.4)) = vertex_cover_number(k22, range_set(Nat.4))
} by {
    k22_is_bipartite
    is_bipartite(k22, range_set(Nat.4))
    k22_maximum_matching_size
    maximum_matching_size(k22, range_set(Nat.4)) = Nat.2
    k22_vertex_cover_number
    vertex_cover_number(k22, range_set(Nat.4)) = Nat.2
    maximum_matching_size(k22, range_set(Nat.4)) = vertex_cover_number(k22, range_set(Nat.4))
}
