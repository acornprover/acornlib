from pair import Pair
from nat import Nat, alt_induction, only_zero_lte_zero, lte_and_lt, lte_trans, lt_suc, lt_trans,
    lte_cancel_suc, add_sub, add_imp_sub, add_assoc, suc_sub_one, sub_zero, lt_imp_lte_suc,
    add_cancels_left, add_cancels_right, sub_lt, lt_cancel_suc, lte_add_right, lt_add_left,
    lt_and_lte, add_comm, add_suc_left, lte_antisymm, lt_suc_right, pos_of_ne_zero, lte_mul_both,
    lte_add_left
from list import List, drop_zero, drop_one, drop_twice, unique_implies_tail_unique,
    singleton_unique, singleton_contains_imp_eq, cons_unique_of_tail_unique_not_contains,
    add_length, add_contains_or, add_contains_left, tail_cancels_cons
from data.basic.witness import choose_or_default, choose_or_default_spec
from finite_set import FiniteSet, fs_remove, fs_insert, fs_from_list, finite_set_subset_contains,
    finite_set_empty_contains_eq, fs_union
from data.finite.finite_set_membership import fs_insert_contains_eq, fs_remove_contains_eq,
    fs_from_list_contains_eq, fs_insert_contains_self, fs_insert_contains_of_contains, fs_remove_subset
from data.finite.finite_set_filter import finite_set_filter_contains_eq
from data.finite.finite_set_product import finite_set_product, finite_set_product_contains_eq
from data.finite.finite_set_card import fs_card, fs_card_empty, fs_card_eq_of_cardinality_is
from data.finite.finite_set_card_ops import fs_card_insert_of_not_contains
from data.finite.finite_set_card_compare import fs_card_mono
from data.finite.finite_set_card_bounds import fs_card_eq_of_mutual_subset
from data.finite.finite_set_card_ops import fs_card_remove_of_contains
from finite_set import finite_set_from_unique_list_cardinality_is_length
from data.finite.finite_set_card_members import fs_two_distinct_members, fs_card_pos_of_contains,
    fs_member_of_card_pos, fs_card_two_of_distinct_members, finite_set_cardinality_has_exact_unique_list
from data.finite.finite_set_subset_intro import fs_subset_eq_intro
from data.finite.finite_set_unique_induction import finite_set_strong_induction
from data.finite.finite_set_nonempty import finite_set_empty_or_inhabited, finite_set_has_member_of_ne_empty
from data.list.list_unique_cons import unique_cons_head_fresh, unique_cons_head_not_in_tail
from data.basic.relation_basic import relation_subset, relation_subset_step, is_symmetric, is_irreflexive
from data.basic.relation import relation_refl_trans_closure, relation_refl_trans_closure_monotone
from graph.simple_graph import SimpleGraph, induced_subgraph, induced_subgraph_adj_eq,
    simple_graph_adj_ne, simple_graph_adj_comm, simple_graph_adj_irreflexive
from graph.simple_graph_tree import is_acyclic, is_acyclic_subset, is_leaf, is_tree,
    is_acyclic_of_card_le_two, simple_graph_cycle, simple_graph_connected_on, tree_has_edge
from graph.simple_graph_connectivity import simple_graph_reachable, simple_graph_reachable_refl,
    simple_graph_reachable_transitive, simple_graph_component
from graph.simple_graph_walks import simple_graph_walk, simple_graph_path, simple_graph_closed_walk,
    simple_graph_walk_cons_iff, simple_graph_walk_nil, simple_graph_walk_of_adj,
    simple_graph_walk_append_adj, simple_graph_walk_vertices, simple_graph_closed_walk_intro,
    simple_graph_closed_walk_is_walk, simple_graph_closed_walk_positive_length,
    simple_graph_walk_imp_reachable, simple_graph_path_walk, simple_graph_path_vertices_unique,
    simple_graph_path_intro
from graph.simple_graph_connectivity import simple_graph_reachable_of_adj
from graph.simple_graph_degree import degree, neighborhood, neighborhood_contains_eq
from graph.simple_graph_edges import directed_edges, directed_edge_count,
    directed_edges_contains_eq, directed_edges_contains_pair, directed_edges_adj,
    directed_edges_ne, directed_edges_first, directed_edges_second, directed_edges_reverse,
    directed_edge_count_eq
from graph.simple_graph_edge_fibers import fs_card_disjoint_union

numerals Nat

/// A forest on the vertex set `s`: an acyclic graph whose vertices are `s`.
///
/// A tree is a connected acyclic graph; a forest drops the connectedness and is
/// just acyclic.
define is_forest[V](g: SimpleGraph[V], s: FiniteSet[V]) -> Bool {
    is_acyclic(g, s)
}

/// A forest is acyclic on its vertex set.
theorem forest_acyclic[V](g: SimpleGraph[V], s: FiniteSet[V]) {
    is_forest(g, s) implies is_acyclic(g, s)
} by {
    if is_forest(g, s) {
        is_forest(g, s) = is_acyclic(g, s)
        is_acyclic(g, s)
    }
}

/// An acyclic graph is a forest.
theorem forest_of_acyclic[V](g: SimpleGraph[V], s: FiniteSet[V]) {
    is_acyclic(g, s) implies is_forest(g, s)
} by {
    if is_acyclic(g, s) {
        is_forest(g, s) = is_acyclic(g, s)
        is_forest(g, s)
    }
}

/// Forest-ness is inherited by smaller vertex sets.
theorem forest_subset[V](g: SimpleGraph[V], s: FiniteSet[V], t: FiniteSet[V]) {
    is_forest(g, s) and t.subset_eq(s) implies is_forest(g, t)
} by {
    if is_forest(g, s) and t.subset_eq(s) {
        forest_acyclic(g, s)
        is_acyclic(g, s)
        is_acyclic_subset(g, s, t)
        is_acyclic(g, t)
        forest_of_acyclic(g, t)
        is_forest(g, t)
    }
}

/// A graph on at most two vertices is a forest: a cycle needs three vertices.
theorem forest_of_card_le_two[V](g: SimpleGraph[V], s: FiniteSet[V]) {
    fs_card(s) <= Nat.2 implies is_forest(g, s)
} by {
    if fs_card(s) <= Nat.2 {
        is_acyclic_of_card_le_two(g, s)
        is_acyclic(g, s)
        forest_of_acyclic(g, s)
        is_forest(g, s)
    }
}

/// A forest with at least two vertices has an edge.
///
/// A forest need not have an edge on two vertices, but with a connectedness
/// hypothesis it does: every tree with at least two vertices has an edge.
theorem forest_has_edge[V](g: SimpleGraph[V], s: FiniteSet[V]) {
    is_forest(g, s) and is_tree(g, s) and Nat.2 <= fs_card(s) implies
        exists(x: V, y: V) {
            s.contains(x) and s.contains(y) and x != y and g.adj(x, y)
        }
} by {
    if is_forest(g, s) and is_tree(g, s) and Nat.2 <= fs_card(s) {
        tree_has_edge(g, s)
        exists(x: V, y: V) {
            s.contains(x) and s.contains(y) and x != y and g.adj(x, y)
        }
    }
}

/// The last vertex of the walk `[start] + steps`.
define walk_last[V](start: V, steps: List[V]) -> V {
    match steps {
        List.nil[V] { start }
        List.cons(head, tail) { walk_last(head, tail) }
    }
}

/// The vertex of `[start] + steps` just before the last one, or `start` if there is none.
define walk_penultimate[V](start: V, steps: List[V]) -> V {
    match steps {
        List.nil[V] { start }
        List.cons(head, tail) {
            match tail {
                List.nil[V] { start }
                List.cons(head2, tail2) { walk_penultimate(head, tail) }
            }
        }
    }
}

/// A simple path in `g` all of whose vertices lie in `s`.
///
/// The finish is implicit: it is the last vertex of `[start] + steps`.
define simple_graph_path_in[V](g: SimpleGraph[V], s: FiniteSet[V], start: V, steps: List[V]) -> Bool {
    simple_graph_path(g, start, walk_last(start, steps), steps) and s.contains(start) and
        (forall(x: V) { steps.contains(x) implies s.contains(x) })
}

/// The finish of a walk is its last vertex.
theorem walk_finish_eq_last[V](g: SimpleGraph[V], a: V, b: V, steps: List[V]) {
    simple_graph_walk(g, a, b, steps) implies b = walk_last(a, steps)
} by {
    define p(xs: List[V]) -> Bool {
        forall(x: V, y: V) {
            simple_graph_walk(g, x, y, xs) implies y = walk_last(x, xs)
        }
    }
    forall(x: V, y: V) {
        if simple_graph_walk(g, x, y, List.nil[V]) {
            simple_graph_walk_nil(g, x, y)
            simple_graph_walk(g, x, y, List.nil[V]) = (x = y)
            x = y
            walk_last(x, List.nil[V]) = x
            y = walk_last(x, List.nil[V])
        }
        p(List.nil[V])
    }
    p(List.nil[V])
    forall(head: V, tail: List[V]) {
        if p(tail) {
            forall(x: V, y: V) {
                if simple_graph_walk(g, x, y, List.cons(head, tail)) {
                    simple_graph_walk_cons_iff(g, x, y, head, tail)
                    simple_graph_walk(g, x, y, List.cons(head, tail)) =
                        (g.adj(x, head) and simple_graph_walk(g, head, y, tail))
                    simple_graph_walk(g, head, y, tail)
                    p(tail) = forall(x2: V, y2: V) {
                        simple_graph_walk(g, x2, y2, tail) implies y2 = walk_last(x2, tail)
                    }
                    y = walk_last(head, tail)
                    walk_last(x, List.cons(head, tail)) = walk_last(head, tail)
                    y = walk_last(x, List.cons(head, tail))
                }
            }
            p(List.cons(head, tail))
        }
    }
    forall(head: V, tail: List[V]) { p(tail) implies p(List.cons(head, tail)) }
    function[T0](x0: List[T0] -> Bool) {
        not forall(x1: T0, x2: List[T0]) { not x0(x2) or x0(List.cons(x1, x2)) } or
        not x0(List.nil[T0]) or forall(x3: List[T0]) { x0(x3) } = true
    }[V](p)
    function[T0](x0: List[T0] -> Bool, x1: List[T0]) {
        not forall(x2: T0, x3: List[T0]) { not x0(x3) or x0(List.cons(x2, x3)) } or
        not x0(List.nil[T0]) or x0(x1)
    }[V](p, steps)
    p(steps)
    p(steps) = forall(x2: V, y2: V) {
        simple_graph_walk(g, x2, y2, steps) implies y2 = walk_last(x2, steps)
    }
    if simple_graph_walk(g, a, b, steps) {
        b = walk_last(a, steps)
    }
}

/// The last vertex of a walk lies on the walk: it is the start or one of the targets.
theorem walk_last_in[V](start: V, steps: List[V]) {
    walk_last(start, steps) = start or steps.contains(walk_last(start, steps))
} by {
    define p(xs: List[V]) -> Bool {
        forall(x: V) {
            walk_last(x, xs) = x or xs.contains(walk_last(x, xs))
        }
    }
    forall(a: V) {
        walk_last(a, List.nil[V]) = a
        walk_last(a, List.nil[V]) = a or List.nil[V].contains(walk_last(a, List.nil[V]))
    }
    p(List.nil[V])
    forall(head: V, tail: List[V]) {
        if p(tail) {
            forall(a: V) {
                p(tail) = forall(x2: V) {
                    walk_last(x2, tail) = x2 or tail.contains(walk_last(x2, tail))
                }
                walk_last(head, tail) = head or tail.contains(walk_last(head, tail))
                walk_last(a, List.cons(head, tail)) = walk_last(head, tail)
                if walk_last(head, tail) = head {
                    walk_last(a, List.cons(head, tail)) = head
                    List.cons(head, tail).contains(head)
                    List.cons(head, tail).contains(walk_last(a, List.cons(head, tail)))
                    walk_last(a, List.cons(head, tail)) = a or List.cons(head, tail).contains(walk_last(a, List.cons(head, tail)))
                }
                if not walk_last(head, tail) = head {
                    tail.contains(walk_last(head, tail))
                    List.cons(head, tail).contains(walk_last(head, tail))
                    walk_last(a, List.cons(head, tail)) = walk_last(head, tail)
                    List.cons(head, tail).contains(walk_last(a, List.cons(head, tail)))
                    walk_last(a, List.cons(head, tail)) = a or List.cons(head, tail).contains(walk_last(a, List.cons(head, tail)))
                }
                walk_last(a, List.cons(head, tail)) = a or List.cons(head, tail).contains(walk_last(a, List.cons(head, tail)))
            }
            p(List.cons(head, tail))
        }
    }
    forall(head: V, tail: List[V]) { p(tail) implies p(List.cons(head, tail)) }
    function[T0](x0: List[T0] -> Bool) {
        not forall(x1: T0, x2: List[T0]) { not x0(x2) or x0(List.cons(x1, x2)) } or
        not x0(List.nil[T0]) or forall(x3: List[T0]) { x0(x3) } = true
    }[V](p)
    function[T0](x0: List[T0] -> Bool, x1: List[T0]) {
        not forall(x2: T0, x3: List[T0]) { not x0(x3) or x0(List.cons(x2, x3)) } or
        not x0(List.nil[T0]) or x0(x1)
    }[V](p, steps)
    p(steps)
    p(steps) = forall(x2: V) {
        walk_last(x2, steps) = x2 or steps.contains(walk_last(x2, steps))
    }
    walk_last(start, steps) = start or steps.contains(walk_last(start, steps))
}

/// A path in a set is a walk.
theorem path_in_walk[V](g: SimpleGraph[V], s: FiniteSet[V], a: V, steps: List[V]) {
    simple_graph_path_in(g, s, a, steps) implies
        simple_graph_walk(g, a, walk_last(a, steps), steps)
} by {
    if simple_graph_path_in(g, s, a, steps) {
        simple_graph_path_in(g, s, a, steps) =
            (simple_graph_path(g, a, walk_last(a, steps), steps) and s.contains(a) and
                (forall(x: V) { steps.contains(x) implies s.contains(x) }))
        simple_graph_path(g, a, walk_last(a, steps), steps)
        simple_graph_path_walk(g, a, walk_last(a, steps), steps)
        simple_graph_walk(g, a, walk_last(a, steps), steps)
    }
}

/// A path in a set is a simple path.
theorem path_in_path[V](g: SimpleGraph[V], s: FiniteSet[V], a: V, steps: List[V]) {
    simple_graph_path_in(g, s, a, steps) implies
        simple_graph_path(g, a, walk_last(a, steps), steps)
} by {
    if simple_graph_path_in(g, s, a, steps) {
        simple_graph_path_in(g, s, a, steps) =
            (simple_graph_path(g, a, walk_last(a, steps), steps) and s.contains(a) and
                (forall(x: V) { steps.contains(x) implies s.contains(x) }))
        simple_graph_path(g, a, walk_last(a, steps), steps)
    }
}

/// The start of a path in a set lies in the set.
theorem path_in_start_in[V](g: SimpleGraph[V], s: FiniteSet[V], a: V, steps: List[V]) {
    simple_graph_path_in(g, s, a, steps) implies s.contains(a)
} by {
    if simple_graph_path_in(g, s, a, steps) {
        simple_graph_path_in(g, s, a, steps) =
            (simple_graph_path(g, a, walk_last(a, steps), steps) and s.contains(a) and
                (forall(x: V) { steps.contains(x) implies s.contains(x) }))
        s.contains(a)
    }
}

/// Every target of a path in a set lies in the set.
theorem path_in_steps_in[V](g: SimpleGraph[V], s: FiniteSet[V], a: V, steps: List[V], x: V) {
    simple_graph_path_in(g, s, a, steps) implies (steps.contains(x) implies s.contains(x))
} by {
    if simple_graph_path_in(g, s, a, steps) {
        simple_graph_path_in(g, s, a, steps) =
            (simple_graph_path(g, a, walk_last(a, steps), steps) and s.contains(a) and
                (forall(x2: V) { steps.contains(x2) implies s.contains(x2) }))
        forall(x2: V) { steps.contains(x2) implies s.contains(x2) }
        steps.contains(x) implies s.contains(x)
        if steps.contains(x) {
            s.contains(x)
        }
    }
}

/// The full vertex list of a path in a set has no repeated vertices.
theorem path_in_vertices_unique[V](g: SimpleGraph[V], s: FiniteSet[V], a: V, steps: List[V]) {
    simple_graph_path_in(g, s, a, steps) implies
        simple_graph_walk_vertices(a, steps).is_unique
} by {
    if simple_graph_path_in(g, s, a, steps) {
        path_in_path(g, s, a, steps)
        simple_graph_path(g, a, walk_last(a, steps), steps)
        simple_graph_path_vertices_unique(g, a, walk_last(a, steps), steps)
        simple_graph_walk_vertices(a, steps).is_unique
    }
}

/// The endpoint of a path in a set lies in the set.
theorem path_in_endpoint_in[V](g: SimpleGraph[V], s: FiniteSet[V], a: V, steps: List[V]) {
    simple_graph_path_in(g, s, a, steps) implies s.contains(walk_last(a, steps))
} by {
    if simple_graph_path_in(g, s, a, steps) {
        walk_last_in(a, steps)
        walk_last(a, steps) = a or steps.contains(walk_last(a, steps))
        path_in_start_in(g, s, a, steps)
        s.contains(a)
        if walk_last(a, steps) = a {
            s.contains(walk_last(a, steps))
        }
        if not walk_last(a, steps) = a {
            steps.contains(walk_last(a, steps))
            path_in_steps_in(g, s, a, steps, walk_last(a, steps))
            s.contains(walk_last(a, steps))
        }
        s.contains(walk_last(a, steps))
    }
}

/// A simple path staying in `s` gives a path in `s`.
theorem path_in_intro[V](g: SimpleGraph[V], s: FiniteSet[V], a: V, steps: List[V]) {
    simple_graph_path(g, a, walk_last(a, steps), steps) and s.contains(a) and
        (forall(x: V) { steps.contains(x) implies s.contains(x) })
        implies simple_graph_path_in(g, s, a, steps)
} by {
    if simple_graph_path(g, a, walk_last(a, steps), steps) and s.contains(a) and
        (forall(x: V) { steps.contains(x) implies s.contains(x) }) {
        simple_graph_path(g, a, walk_last(a, steps), steps) and s.contains(a) and
            (forall(x: V) { steps.contains(x) implies s.contains(x) })
        simple_graph_path_in(g, s, a, steps) =
            (simple_graph_path(g, a, walk_last(a, steps), steps) and s.contains(a) and
                (forall(x: V) { steps.contains(x) implies s.contains(x) }))
        simple_graph_path_in(g, s, a, steps)
    }
}

/// The penultimate vertex is adjacent to the finish of a walk of positive length.
theorem walk_penultimate_adj[V](g: SimpleGraph[V], a: V, b: V, steps: List[V]) {
    simple_graph_walk(g, a, b, steps) and Nat.1 <= steps.length implies
        g.adj(walk_penultimate(a, steps), b)
} by {
    define p(xs: List[V]) -> Bool {
        forall(x: V, y: V) {
            simple_graph_walk(g, x, y, xs) implies
                (xs.length = Nat.0 or g.adj(walk_penultimate(x, xs), y))
        }
    }
    forall(x: V, y: V) {
        if simple_graph_walk(g, x, y, List.nil[V]) {
            simple_graph_walk_nil(g, x, y)
            simple_graph_walk(g, x, y, List.nil[V]) = (x = y)
            x = y
            List.nil[V].length = Nat.0
            List.nil[V].length = Nat.0 or g.adj(walk_penultimate(x, List.nil[V]), y)
        }
        p(List.nil[V])
    }
    p(List.nil[V])
    forall(head: V, tail: List[V]) {
        if p(tail) {
            forall(x: V, y: V) {
                if simple_graph_walk(g, x, y, List.cons(head, tail)) {
                    simple_graph_walk_cons_iff(g, x, y, head, tail)
                    simple_graph_walk(g, x, y, List.cons(head, tail)) =
                        (g.adj(x, head) and simple_graph_walk(g, head, y, tail))
                    simple_graph_walk(g, head, y, tail)
                    match tail {
                        List.nil[V] {
                            tail = List.nil[V]
                            simple_graph_walk(g, head, y, List.nil[V])
                            simple_graph_walk_nil(g, head, y)
                            simple_graph_walk(g, head, y, List.nil[V]) = (head = y)
                            head = y
                            g.adj(x, head)
                            g.adj(x, y)
                            walk_penultimate(x, List.cons(head, List.nil[V])) = x
                            walk_penultimate(x, List.cons(head, tail)) = x
                            List.cons(head, tail).length = tail.length + Nat.1
                            List.cons(head, tail).length = Nat.0 or g.adj(walk_penultimate(x, List.cons(head, tail)), y)
                        }
                        List.cons(head2, tail2) {
                            tail = List.cons(head2, tail2)
                            p(tail) = forall(x2: V, y2: V) {
                                simple_graph_walk(g, x2, y2, tail) implies
                                    (tail.length = Nat.0 or g.adj(walk_penultimate(x2, tail), y2))
                            }
                            tail.length = Nat.0 or g.adj(walk_penultimate(head, tail), y)
                            walk_penultimate(x, List.cons(head, List.cons(head2, tail2))) = walk_penultimate(head, List.cons(head2, tail2))
                            walk_penultimate(x, List.cons(head, tail)) = walk_penultimate(head, tail)
                            if tail.length = Nat.0 {
                                List.cons(head2, tail2).length = tail2.length + Nat.1
                                tail.length = tail2.length + Nat.1
                                tail.length = Nat.0
                                tail2.length + Nat.1 = Nat.0
                                false
                            }
                            not (tail.length = Nat.0)
                            g.adj(walk_penultimate(head, tail), y)
                            List.cons(head, tail).length = tail.length + Nat.1
                            List.cons(head, tail).length = Nat.0 or g.adj(walk_penultimate(x, List.cons(head, tail)), y)
                        }
                    }
                    List.cons(head, tail).length = Nat.0 or g.adj(walk_penultimate(x, List.cons(head, tail)), y)
                }
            }
            p(List.cons(head, tail))
        }
    }
    forall(head: V, tail: List[V]) { p(tail) implies p(List.cons(head, tail)) }
    function[T0](x0: List[T0] -> Bool) {
        not forall(x1: T0, x2: List[T0]) { not x0(x2) or x0(List.cons(x1, x2)) } or
        not x0(List.nil[T0]) or forall(x3: List[T0]) { x0(x3) } = true
    }[V](p)
    function[T0](x0: List[T0] -> Bool, x1: List[T0]) {
        not forall(x2: T0, x3: List[T0]) { not x0(x3) or x0(List.cons(x2, x3)) } or
        not x0(List.nil[T0]) or x0(x1)
    }[V](p, steps)
    p(steps)
    p(steps) = forall(x2: V, y2: V) {
        simple_graph_walk(g, x2, y2, steps) implies
            (steps.length = Nat.0 or g.adj(walk_penultimate(x2, steps), y2))
    }
    steps.length = Nat.0 or g.adj(walk_penultimate(a, steps), b)
    if steps.length = Nat.0 {
        Nat.1 <= steps.length
        Nat.1 <= Nat.0
        only_zero_lte_zero(Nat.1)
        Nat.1 <= Nat.0 implies Nat.1 = Nat.0
        Nat.1 = Nat.0
        false
    }
    not (steps.length = Nat.0)
    g.adj(walk_penultimate(a, steps), b)
}

/// The penultimate vertex lies on the walk: it is the start or one of the targets.
theorem walk_penultimate_vertex[V](start: V, steps: List[V]) {
    walk_penultimate(start, steps) = start or steps.contains(walk_penultimate(start, steps))
} by {
    define p(xs: List[V]) -> Bool {
        forall(x: V) {
            walk_penultimate(x, xs) = x or xs.contains(walk_penultimate(x, xs))
        }
    }
    forall(x: V) {
        walk_penultimate(x, List.nil[V]) = x
        walk_penultimate(x, List.nil[V]) = x or List.nil[V].contains(walk_penultimate(x, List.nil[V]))
    }
    p(List.nil[V])
    forall(head: V, tail: List[V]) {
        if p(tail) {
            forall(x: V) {
                p(tail) = forall(x2: V) {
                    walk_penultimate(x2, tail) = x2 or tail.contains(walk_penultimate(x2, tail))
                }
                walk_penultimate(head, tail) = head or tail.contains(walk_penultimate(head, tail))
                match tail {
                    List.nil[V] {
                        tail = List.nil[V]
                        walk_penultimate(x, List.cons(head, List.nil[V])) = x
                        walk_penultimate(x, List.cons(head, tail)) = x
                        walk_penultimate(x, List.cons(head, tail)) = x or List.cons(head, tail).contains(walk_penultimate(x, List.cons(head, tail)))
                    }
                    List.cons(head2, tail2) {
                        tail = List.cons(head2, tail2)
                        walk_penultimate(x, List.cons(head, List.cons(head2, tail2))) = walk_penultimate(head, List.cons(head2, tail2))
                        walk_penultimate(x, List.cons(head, tail)) = walk_penultimate(head, tail)
                        if walk_penultimate(head, tail) = head {
                            List.cons(head, tail).contains(head)
                            List.cons(head, tail).contains(walk_penultimate(x, List.cons(head, tail)))
                            walk_penultimate(x, List.cons(head, tail)) = x or List.cons(head, tail).contains(walk_penultimate(x, List.cons(head, tail)))
                        }
                        if not walk_penultimate(head, tail) = head {
                            tail.contains(walk_penultimate(head, tail))
                            List.cons(head, tail).contains(walk_penultimate(head, tail))
                            List.cons(head, tail).contains(walk_penultimate(x, List.cons(head, tail)))
                            walk_penultimate(x, List.cons(head, tail)) = x or List.cons(head, tail).contains(walk_penultimate(x, List.cons(head, tail)))
                        }
                        walk_penultimate(x, List.cons(head, tail)) = x or List.cons(head, tail).contains(walk_penultimate(x, List.cons(head, tail)))
                    }
                }
                walk_penultimate(x, List.cons(head, tail)) = x or List.cons(head, tail).contains(walk_penultimate(x, List.cons(head, tail)))
            }
            p(List.cons(head, tail))
        }
    }
    forall(head: V, tail: List[V]) { p(tail) implies p(List.cons(head, tail)) }
    function[T0](x0: List[T0] -> Bool) {
        not forall(x1: T0, x2: List[T0]) { not x0(x2) or x0(List.cons(x1, x2)) } or
        not x0(List.nil[T0]) or forall(x3: List[T0]) { x0(x3) } = true
    }[V](p)
    function[T0](x0: List[T0] -> Bool, x1: List[T0]) {
        not forall(x2: T0, x3: List[T0]) { not x0(x3) or x0(List.cons(x2, x3)) } or
        not x0(List.nil[T0]) or x0(x1)
    }[V](p, steps)
    p(steps)
    p(steps) = forall(x2: V) {
        walk_penultimate(x2, steps) = x2 or steps.contains(walk_penultimate(x2, steps))
    }
    walk_penultimate(start, steps) = start or steps.contains(walk_penultimate(start, steps))
}

/// Appending a fresh element keeps a list without repeats.
theorem append_unique[T](xs: List[T], u: T) {
    xs.is_unique and not xs.contains(u) implies xs.append(u).is_unique
} by {
    define p(ys: List[T]) -> Bool {
        ys.is_unique and not ys.contains(u) implies ys.append(u).is_unique
    }
    if List.nil[T].is_unique and not List.nil[T].contains(u) {
        List.nil[T].append(u) = List.singleton(u)
        singleton_unique(u)
        List.singleton(u).is_unique
        List.nil[T].append(u).is_unique
    }
    p(List.nil[T])
    forall(head: T, tail: List[T]) {
        if p(tail) {
            if List.cons(head, tail).is_unique and not List.cons(head, tail).contains(u) {
                unique_implies_tail_unique(head, tail)
                tail.is_unique
                unique_cons_head_not_in_tail(head, tail)
                not tail.contains(head)
                List.cons(head, tail).contains(u) = (u = head or tail.contains(u))
                not (u = head or tail.contains(u))
                not tail.contains(u)
                p(tail)
                tail.append(u).is_unique
                List.cons(head, tail).append(u) = List.cons(head, tail.append(u))
                tail.append(u) = tail + List.singleton(u)
                add_contains_or(tail, List.singleton(u), head)
                (tail + List.singleton(u)).contains(head) = (tail.contains(head) or List.singleton(u).contains(head))
                tail.append(u).contains(head) = (tail.contains(head) or List.singleton(u).contains(head))
                not tail.contains(head)
                if List.singleton(u).contains(head) {
                    singleton_contains_imp_eq(u, head)
                    head = u
                    u = head
                    false
                }
                not tail.append(u).contains(head)
                cons_unique_of_tail_unique_not_contains(head, tail.append(u))
                List.cons(head, tail.append(u)).is_unique
                List.cons(head, tail).append(u).is_unique
            }
            p(List.cons(head, tail))
        }
    }
    forall(head: T, tail: List[T]) { p(tail) implies p(List.cons(head, tail)) }
    function[T0](x0: List[T0] -> Bool) {
        not forall(x1: T0, x2: List[T0]) { not x0(x2) or x0(List.cons(x1, x2)) } or
        not x0(List.nil[T0]) or forall(x3: List[T0]) { x0(x3) } = true
    }[T](p)
    function[T0](x0: List[T0] -> Bool, x1: List[T0]) {
        not forall(x2: T0, x3: List[T0]) { not x0(x3) or x0(List.cons(x2, x3)) } or
        not x0(List.nil[T0]) or x0(x1)
    }[T](p, xs)
    p(xs)
    if xs.is_unique and not xs.contains(u) {
        xs.append(u).is_unique
    }
}

/// Extending a path in a set by a fresh adjacent vertex keeps it a path in the set.
theorem path_in_extend[V](g: SimpleGraph[V], s: FiniteSet[V], a: V, steps: List[V], u: V) {
    simple_graph_path_in(g, s, a, steps) and s.contains(u) and g.adj(walk_last(a, steps), u) and
        u != a and not steps.contains(u)
        implies simple_graph_path_in(g, s, a, steps.append(u))
} by {
    if simple_graph_path_in(g, s, a, steps) and s.contains(u) and g.adj(walk_last(a, steps), u) and
        u != a and not steps.contains(u) {
        path_in_walk(g, s, a, steps)
        simple_graph_walk(g, a, walk_last(a, steps), steps)
        simple_graph_walk_append_adj(g, a, walk_last(a, steps), u, steps)
        simple_graph_walk(g, a, u, steps.append(u))
        simple_graph_walk_vertices(a, steps.append(u)) = List.cons(a, steps.append(u))
        List.cons(a, steps).append(u) = List.cons(a, steps.append(u))
        simple_graph_walk_vertices(a, steps) = List.cons(a, steps)
        path_in_vertices_unique(g, s, a, steps)
        List.cons(a, steps).is_unique
        append_unique(List.cons(a, steps), u)
        List.cons(a, steps).contains(u) = (u = a or steps.contains(u))
        if List.cons(a, steps).contains(u) {
            u = a or steps.contains(u)
            if u = a {
                u != a
                false
            }
            if steps.contains(u) {
                not steps.contains(u)
                false
            }
            false
        }
        not List.cons(a, steps).contains(u)
        List.cons(a, steps).append(u).is_unique
        simple_graph_walk_vertices(a, steps.append(u)).is_unique
        walk_last(a, steps.append(u)) = u
        simple_graph_path_intro(g, a, u, steps.append(u))
        simple_graph_path(g, a, u, steps.append(u))
        path_in_start_in(g, s, a, steps)
        s.contains(a)
        forall(x: V) {
            if steps.append(u).contains(x) {
                steps.append(u).contains(x) = (steps.contains(x) or List.singleton(u).contains(x))
                if steps.contains(x) {
                    path_in_steps_in(g, s, a, steps, x)
                    s.contains(x)
                }
                if List.singleton(u).contains(x) {
                    singleton_contains_imp_eq(u, x)
                    x = u
                    s.contains(x)
                }
                s.contains(x)
            }
            steps.append(u).contains(x) implies s.contains(x)
        }
        forall(x: V) { steps.append(u).contains(x) implies s.contains(x) }
        walk_last(a, steps.append(u)) = u
        simple_graph_path(g, a, u, steps.append(u))
        simple_graph_path(g, a, walk_last(a, steps.append(u)), steps.append(u))
        simple_graph_path(g, a, walk_last(a, steps.append(u)), steps.append(u)) and s.contains(a) and
            (forall(x: V) { steps.append(u).contains(x) implies s.contains(x) })
        path_in_intro(g, s, a, steps.append(u))
        simple_graph_path_in(g, s, a, steps.append(u))
    }
}

/// Dropping from a cons list reduces to dropping from the tail.
theorem drop_cons[T](h: T, t: List[T], n: Nat) {
    List.cons(h, t).drop(n + Nat.1) = t.drop(n)
} by {
    n + Nat.1 = n.suc
    tail_cancels_cons(h, t)
    List.cons(h, t).tail = t
    List.cons(h, t).drop(n.suc) = List.cons(h, t).tail.drop(n)
    List.cons(h, t).drop(n.suc) = t.drop(n)
    List.cons(h, t).drop(n + Nat.1) = t.drop(n)
}

/// Dropping from the empty list keeps it empty.
theorem drop_nil[T](n: Nat) {
    List.nil[T].drop(n) = List.nil[T]
} by {
    define p(k: Nat) -> Bool {
        List.nil[T].drop(k) = List.nil[T]
    }
    List.nil[T].drop(Nat.0) = List.nil[T]
    p(Nat.0)
    forall(k: Nat) {
        if p(k) {
            List.nil[T].tail = List.nil[T]
            List.nil[T].drop(k.suc) = List.nil[T].tail.drop(k)
            List.nil[T].drop(k.suc) = List.nil[T].drop(k)
            List.nil[T].drop(k.suc) = List.nil[T]
            p(k.suc)
        }
    }
    forall(k: Nat) { p(k) implies p(k.suc) }
    p(Nat.0) and forall(k: Nat) { p(k) implies p(k.suc) }
    alt_induction(p)
    forall(k: Nat) { p(k) }
    p(n)
    List.nil[T].drop(n) = List.nil[T]
}

/// (a + 1) - (b + 1) = a - b when b <= a.
theorem sub_suc_suc(a: Nat, b: Nat) {
    b <= a implies (a + Nat.1) - (b + Nat.1) = a - b
} by {
    if b <= a {
        add_sub(a, b)
        a - b + b = a
        (a - b + b) + Nat.1 = a + Nat.1
        add_assoc(a - b, b, Nat.1)
        (a - b + b) + Nat.1 = (a - b) + (b + Nat.1)
        (a - b) + (b + Nat.1) = a + Nat.1
        add_imp_sub(a - b, b + Nat.1, a + Nat.1)
        (a + Nat.1) - (b + Nat.1) = a - b
    }
}

/// Dropping preserves the uniqueness of a list.
theorem drop_preserves_unique[T](xs: List[T], n: Nat) {
    xs.is_unique implies xs.drop(n).is_unique
} by {
    define p(k: Nat) -> Bool {
        forall(ys: List[T]) { ys.is_unique implies ys.drop(k).is_unique }
    }
    forall(ys: List[T]) {
        ys.drop(Nat.0) = ys
        ys.is_unique implies ys.drop(Nat.0).is_unique
    }
    p(Nat.0)
    forall(k: Nat) {
        if p(k) {
            forall(ys: List[T]) {
                if ys.is_unique {
                    match ys {
                        List.nil[T] {
                            ys = List.nil[T]
                            drop_nil[T](k + Nat.1)
                            List.nil[T].drop(k + Nat.1) = List.nil[T]
                            ys.drop(k + Nat.1) = List.nil[T]
                            List.nil[T].is_unique
                            ys.drop(k + Nat.1).is_unique
                        }
                        List.cons(h, t) {
                            ys = List.cons(h, t)
                            unique_implies_tail_unique(h, t)
                            t.is_unique
                            p(k) = forall(ys2: List[T]) { ys2.is_unique implies ys2.drop(k).is_unique }
                            t.drop(k).is_unique
                            drop_cons(h, t, k)
                            List.cons(h, t).drop(k + Nat.1) = t.drop(k)
                            ys.drop(k + Nat.1) = List.cons(h, t).drop(k + Nat.1)
                            ys.drop(k + Nat.1) = t.drop(k)
                            ys.drop(k + Nat.1).is_unique
                        }
                    }
                }
                ys.is_unique implies ys.drop(k + Nat.1).is_unique
            }
            p(k + Nat.1)
        }
    }
    forall(k: Nat) { p(k) implies p(k + Nat.1) }
    n + Nat.1 = n.suc
    forall(k: Nat) { p(k) implies p(k.suc) }
    p(Nat.0) and forall(k: Nat) { p(k) implies p(k.suc) }
    alt_induction(p)
    forall(k: Nat) { p(k) }
    p(n)
    p(n) = forall(ys: List[T]) { ys.is_unique implies ys.drop(n).is_unique }
    if xs.is_unique {
        xs.drop(n).is_unique
    }
}

/// Dropping cannot create a member.
theorem drop_contains[T](xs: List[T], n: Nat, y: T) {
    xs.drop(n).contains(y) implies xs.contains(y)
} by {
    define p(k: Nat) -> Bool {
        forall(ys: List[T], z: T) { ys.drop(k).contains(z) implies ys.contains(z) }
    }
    forall(ys: List[T], z: T) {
        ys.drop(Nat.0) = ys
        ys.drop(Nat.0).contains(z) implies ys.contains(z)
    }
    p(Nat.0)
    forall(k: Nat) {
        if p(k) {
            forall(ys: List[T], z: T) {
                if ys.drop(k + Nat.1).contains(z) {
                    match ys {
                        List.nil[T] {
                            ys = List.nil[T]
                            drop_nil[T](k + Nat.1)
                            List.nil[T].drop(k + Nat.1) = List.nil[T]
                            ys.drop(k + Nat.1) = List.nil[T]
                            ys.drop(k + Nat.1).contains(z) = List.nil[T].contains(z)
                            List.nil[T].contains(z) = false
                            false
                        }
                        List.cons(h, t) {
                            ys = List.cons(h, t)
                            drop_cons(h, t, k)
                            List.cons(h, t).drop(k + Nat.1) = t.drop(k)
                            ys.drop(k + Nat.1) = List.cons(h, t).drop(k + Nat.1)
                            ys.drop(k + Nat.1) = t.drop(k)
                            t.drop(k).contains(z)
                            p(k) = forall(ys2: List[T], z2: T) { ys2.drop(k).contains(z2) implies ys2.contains(z2) }
                            t.contains(z)
                            List.cons(h, t).contains(z) = (z = h or t.contains(z))
                            List.cons(h, t).contains(z)
                            ys.contains(z)
                        }
                    }
                    ys.contains(z)
                }
                ys.drop(k + Nat.1).contains(z) implies ys.contains(z)
            }
            p(k + Nat.1)
        }
    }
    forall(k: Nat) { p(k) implies p(k + Nat.1) }
    n + Nat.1 = n.suc
    forall(k: Nat) { p(k) implies p(k.suc) }
    p(Nat.0) and forall(k: Nat) { p(k) implies p(k.suc) }
    alt_induction(p)
    forall(k: Nat) { p(k) }
    p(n)
    p(n) = forall(ys: List[T], z: T) { ys.drop(n).contains(z) implies ys.contains(z) }
    if xs.drop(n).contains(y) {
        xs.contains(y)
    }
}

/// The length of a dropped list plus the dropped amount.
define drop_len_plus[T](xs: List[T], n: Nat) -> Nat {
    xs.drop(n).length + n
}

/// Dropping shortens a list by exactly the dropped amount.
///
/// Stated with addition on the left so the induction stays free of subtraction.
theorem drop_length[T](xs: List[T], n: Nat) {
    n <= xs.length implies xs.length = drop_len_plus(xs, n)
} by {
    define p(k: Nat) -> Bool {
        forall(ys: List[T]) { k <= ys.length implies ys.length = drop_len_plus(ys, k) }
    }
    forall(ys: List[T]) {
        ys.drop(Nat.0) = ys
        drop_len_plus(ys, Nat.0) = ys.drop(Nat.0).length + Nat.0
        drop_len_plus(ys, Nat.0) = ys.length + Nat.0
        drop_len_plus(ys, Nat.0) = ys.length
        if Nat.0 <= ys.length {
            ys.length = drop_len_plus(ys, Nat.0)
        }
    }
    p(Nat.0)
    forall(k: Nat) {
        if p(k) {
            forall(ys: List[T]) {
                if k + Nat.1 <= ys.length {
                    match ys {
                        List.nil[T] {
                            ys = List.nil[T]
                            List.nil[T].length = Nat.0
                            k + Nat.1 <= Nat.0
                            k + Nat.1 = k.suc
                            k.suc <= Nat.0
                            only_zero_lte_zero(k.suc)
                            k.suc = Nat.0
                            false
                        }
                        List.cons(h, t) {
                            ys = List.cons(h, t)
                            List.cons(h, t).length = t.length + Nat.1
                            k + Nat.1 <= t.length + Nat.1
                            k + Nat.1 = k.suc
                            t.length + Nat.1 = t.length.suc
                            k.suc <= t.length.suc
                            lte_cancel_suc(k, t.length)
                            k <= t.length
                            p(k) = forall(ys2: List[T]) { k <= ys2.length implies ys2.length = drop_len_plus(ys2, k) }
                            t.length = drop_len_plus(t, k)
                            drop_len_plus(t, k) = t.drop(k).length + k
                            drop_cons(h, t, k)
                            List.cons(h, t).drop(k + Nat.1) = t.drop(k)
                            drop_len_plus(List.cons(h, t), k + Nat.1) = List.cons(h, t).drop(k + Nat.1).length + (k + Nat.1)
                            drop_len_plus(List.cons(h, t), k + Nat.1) = t.drop(k).length + (k + Nat.1)
                            t.length = t.drop(k).length + k
                            t.length + Nat.1 = t.drop(k).length + k + Nat.1
                            add_assoc(t.drop(k).length, k, Nat.1)
                            t.length + Nat.1 = t.drop(k).length + (k + Nat.1)
                            drop_len_plus(List.cons(h, t), k + Nat.1) = t.length + Nat.1
                            List.cons(h, t).length = drop_len_plus(List.cons(h, t), k + Nat.1)
                            ys.length = List.cons(h, t).length
                            ys.length = drop_len_plus(List.cons(h, t), k + Nat.1)
                            drop_len_plus(ys, k + Nat.1) = drop_len_plus(List.cons(h, t), k + Nat.1)
                            ys.length = drop_len_plus(ys, k + Nat.1)
                        }
                    }
                    ys.length = drop_len_plus(ys, k + Nat.1)
                }
                k + Nat.1 <= ys.length implies ys.length = drop_len_plus(ys, k + Nat.1)
            }
            p(k + Nat.1)
        }
    }
    forall(k: Nat) { p(k) implies p(k + Nat.1) }
    n + Nat.1 = n.suc
    forall(k: Nat) { p(k) implies p(k.suc) }
    p(Nat.0) and forall(k: Nat) { p(k) implies p(k.suc) }
    alt_induction(p)
    forall(k: Nat) { p(k) }
    p(n)
    p(n) = forall(ys: List[T]) { n <= ys.length implies ys.length = drop_len_plus(ys, n) }
    n <= xs.length implies xs.length = drop_len_plus(xs, n)
}

/// Indexing into a cons list reduces to indexing into the tail.
theorem get_idx_cons[T](h: T, t: List[T], n: Nat) {
    List.cons(h, t).get_idx(n + Nat.1) = t.get_idx(n)
} by {
    n + Nat.1 = n.suc
    lt_suc(n)
    n < n.suc
    lte_and_lt(Nat.0, n, n.suc)
    Nat.0 < n.suc
    n.suc > Nat.0
    List.cons(h, t).get_idx(n.suc) = t.get_idx(n.suc - Nat.1)
    suc_sub_one(n)
    n.suc - Nat.1 = n
    List.cons(h, t).get_idx(n.suc) = t.get_idx(n)
    List.cons(h, t).get_idx(n + Nat.1) = t.get_idx(n)
}

/// The first occurrence index of a non-head item lives one deeper in the tail.
theorem find_first_idx_cons[T](head: T, tail: List[T], item: T) {
    head != item implies List.cons(head, tail).find_first_idx(item) = Nat.1 + tail.find_first_idx(item)
} by {
    if head != item {
        not (head = item)
        List.cons(head, tail).find_first_idx(item) = Nat.1 + tail.find_first_idx(item)
    }
}

/// The first occurrence index really points at the item.
theorem find_first_idx_works[T](xs: List[T], item: T) {
    xs.contains(item) implies xs.get_idx(xs.find_first_idx(item)) = Option.some(item)
} by {
    define p(ys: List[T]) -> Bool {
        ys.get_idx(ys.find_first_idx(item)) = Option.some(item) or not ys.contains(item)
    }
    if List.nil[T].contains(item) {
        List.nil[T].contains(item) = false
        false
    }
    not List.nil[T].contains(item)
    p(List.nil[T]) = (List.nil[T].get_idx(List.nil[T].find_first_idx(item)) = Option.some(item) or not List.nil[T].contains(item))
    p(List.nil[T])
    forall(head: T, tail: List[T]) {
        if p(tail) {
            if head = item {
                List.cons(head, tail).find_first_idx(item) = Nat.0
                List.cons(head, tail).get_idx(Nat.0) = Option.some(head)
                List.cons(head, tail).get_idx(List.cons(head, tail).find_first_idx(item)) = Option.some(head)
                List.cons(head, tail).get_idx(List.cons(head, tail).find_first_idx(item)) = Option.some(item)
            }
            if not head = item {
                head != item
                find_first_idx_cons(head, tail, item)
                List.cons(head, tail).find_first_idx(item) = Nat.1 + tail.find_first_idx(item)
                p(tail) = (tail.get_idx(tail.find_first_idx(item)) = Option.some(item) or not tail.contains(item))
                tail.get_idx(tail.find_first_idx(item)) = Option.some(item) or not tail.contains(item)
                if tail.get_idx(tail.find_first_idx(item)) = Option.some(item) {
                    get_idx_cons(head, tail, tail.find_first_idx(item))
                    List.cons(head, tail).get_idx(tail.find_first_idx(item) + Nat.1) = tail.get_idx(tail.find_first_idx(item))
                    List.cons(head, tail).get_idx(tail.find_first_idx(item) + Nat.1) = Option.some(item)
                    List.cons(head, tail).get_idx(List.cons(head, tail).find_first_idx(item)) = Option.some(item)
                }
                if not tail.get_idx(tail.find_first_idx(item)) = Option.some(item) {
                    not tail.contains(item)
                    List.cons(head, tail).contains(item) = (item = head or tail.contains(item))
                    not (item = head)
                    if List.cons(head, tail).contains(item) {
                        item = head or tail.contains(item)
                        if item = head {
                            item = head
                            not (item = head)
                            false
                        }
                        if tail.contains(item) {
                            not tail.contains(item)
                            false
                        }
                        false
                    }
                    not List.cons(head, tail).contains(item)
                    List.cons(head, tail).get_idx(List.cons(head, tail).find_first_idx(item)) = Option.some(item) or not List.cons(head, tail).contains(item)
                }
                List.cons(head, tail).get_idx(List.cons(head, tail).find_first_idx(item)) = Option.some(item) or not List.cons(head, tail).contains(item)
            }
            List.cons(head, tail).get_idx(List.cons(head, tail).find_first_idx(item)) = Option.some(item) or not List.cons(head, tail).contains(item)
            p(List.cons(head, tail))
        }
    }
    forall(head: T, tail: List[T]) { p(tail) implies p(List.cons(head, tail)) }
    function[T0](x0: List[T0] -> Bool) {
        not forall(x1: T0, x2: List[T0]) { not x0(x2) or x0(List.cons(x1, x2)) } or
        not x0(List.nil[T0]) or forall(x3: List[T0]) { x0(x3) } = true
    }[T](p)
    function[T0](x0: List[T0] -> Bool, x1: List[T0]) {
        not forall(x2: T0, x3: List[T0]) { not x0(x3) or x0(List.cons(x2, x3)) } or
        not x0(List.nil[T0]) or x0(x1)
    }[T](p, xs)
    p(xs)
    p(xs) = (xs.get_idx(xs.find_first_idx(item)) = Option.some(item) or not xs.contains(item))
    xs.get_idx(xs.find_first_idx(item)) = Option.some(item) or not xs.contains(item)
    if xs.contains(item) {
        if not xs.contains(item) {
            false
        }
        xs.get_idx(xs.find_first_idx(item)) = Option.some(item)
    }
    xs.contains(item) implies xs.get_idx(xs.find_first_idx(item)) = Option.some(item)
}

/// The greedy extension guard: a fresh vertex adjacent to the current endpoint.
define greedy_guard[V](g: SimpleGraph[V], s: FiniteSet[V], a: V, steps: List[V], u: V) -> Bool {
    s.contains(u) and g.adj(walk_last(a, steps), u) and u != a and not steps.contains(u)
}

/// The vertex chosen by the greedy extension, when one exists.
define greedy_chosen[V](g: SimpleGraph[V], s: FiniteSet[V], a: V, steps: List[V]) -> V {
    choose_or_default(function(u: V) { greedy_guard(g, s, a, steps, u) }, a)
}

/// A path extended greedily while a fresh neighbor of the endpoint exists.
///
/// The `fuel` budget bounds the number of extensions; the walk is extended only
/// while a fresh adjacent vertex inside `s` exists.
define greedy_path[V](g: SimpleGraph[V], s: FiniteSet[V], a: V, steps: List[V], fuel: Nat) -> List[V] {
    match fuel {
        Nat.zero { steps }
        Nat.suc(f) {
            if exists(u: V) { greedy_guard(g, s, a, steps, u) } {
                greedy_path(g, s, a, steps.append(greedy_chosen(g, s, a, steps)), f)
            } else {
                steps
            }
        }
    }
}

/// The greedy extension keeps a path in a set a path in the set.
theorem greedy_path_preserves_path[V](g: SimpleGraph[V], s: FiniteSet[V], a: V, steps: List[V], fuel: Nat) {
    simple_graph_path_in(g, s, a, steps) implies
        simple_graph_path_in(g, s, a, greedy_path(g, s, a, steps, fuel))
} by {
    define p(f: Nat) -> Bool {
        forall(a2: V, steps2: List[V]) {
            simple_graph_path_in(g, s, a2, steps2) implies
                simple_graph_path_in(g, s, a2, greedy_path(g, s, a2, steps2, f))
        }
    }
    forall(a2: V, steps2: List[V]) {
        if simple_graph_path_in(g, s, a2, steps2) {
            greedy_path(g, s, a2, steps2, Nat.0) = steps2
            simple_graph_path_in(g, s, a2, greedy_path(g, s, a2, steps2, Nat.0))
        }
    }
    p(Nat.0)
    forall(f: Nat) {
        if p(f) {
            forall(a2: V, steps2: List[V]) {
                if simple_graph_path_in(g, s, a2, steps2) {
                    if exists(u: V) { greedy_guard(g, s, a2, steps2, u) } {
                        choose_or_default_spec[V](function(u: V) { greedy_guard(g, s, a2, steps2, u) }, a2)
                        greedy_guard(g, s, a2, steps2, greedy_chosen(g, s, a2, steps2))
                        greedy_guard(g, s, a2, steps2, greedy_chosen(g, s, a2, steps2)) =
                            (s.contains(greedy_chosen(g, s, a2, steps2)) and
                                g.adj(walk_last(a2, steps2), greedy_chosen(g, s, a2, steps2)) and
                                greedy_chosen(g, s, a2, steps2) != a2 and
                                not steps2.contains(greedy_chosen(g, s, a2, steps2)))
                        s.contains(greedy_chosen(g, s, a2, steps2))
                        g.adj(walk_last(a2, steps2), greedy_chosen(g, s, a2, steps2))
                        greedy_chosen(g, s, a2, steps2) != a2
                        not steps2.contains(greedy_chosen(g, s, a2, steps2))
                        path_in_extend(g, s, a2, steps2, greedy_chosen(g, s, a2, steps2))
                        simple_graph_path_in(g, s, a2, steps2.append(greedy_chosen(g, s, a2, steps2)))
                        p(f) = forall(a3: V, steps3: List[V]) {
                            simple_graph_path_in(g, s, a3, steps3) implies
                                simple_graph_path_in(g, s, a3, greedy_path(g, s, a3, steps3, f))
                        }
                        p(f)
                        simple_graph_path_in(g, s, a2, greedy_path(g, s, a2, steps2.append(greedy_chosen(g, s, a2, steps2)), f))
                        greedy_path(g, s, a2, steps2, f.suc) =
                            greedy_path(g, s, a2, steps2.append(greedy_chosen(g, s, a2, steps2)), f)
                        simple_graph_path_in(g, s, a2, greedy_path(g, s, a2, steps2, f.suc))
                    } else {
                        not exists(u: V) { greedy_guard(g, s, a2, steps2, u) }
                        greedy_path(g, s, a2, steps2, f.suc) = steps2
                        simple_graph_path_in(g, s, a2, greedy_path(g, s, a2, steps2, f.suc))
                    }
                    simple_graph_path_in(g, s, a2, greedy_path(g, s, a2, steps2, f.suc))
                }
            }
            p(f.suc)
        }
    }
    forall(f: Nat) { p(f) implies p(f.suc) }
    p(Nat.0) and forall(f: Nat) { p(f) implies p(f.suc) }
    alt_induction(p)
    forall(f: Nat) { p(f) }
    p(fuel)
    p(fuel) = forall(a3: V, steps3: List[V]) {
        simple_graph_path_in(g, s, a3, steps3) implies
            simple_graph_path_in(g, s, a3, greedy_path(g, s, a3, steps3, fuel))
    }
    if simple_graph_path_in(g, s, a, steps) {
        simple_graph_path_in(g, s, a, greedy_path(g, s, a, steps, fuel))
    }
}

/// Appending a single element adds one to the length.
theorem append_length_one[T](xs: List[T], u: T) {
    xs.append(u).length = xs.length + Nat.1
} by {
    define p(ys: List[T]) -> Bool {
        ys.append(u).length = ys.length + Nat.1
    }
    List.nil[T].append(u) = List.singleton(u)
    List.singleton(u) = List.cons(u, List.nil[T])
    List.cons(u, List.nil[T]).length = List.nil[T].length + Nat.1
    List.nil[T].length = Nat.0
    List.singleton(u).length = Nat.1
    List.nil[T].append(u).length = Nat.1
    List.nil[T].append(u).length = List.nil[T].length + Nat.1
    p(List.nil[T])
    forall(head: T, tail: List[T]) {
        if p(tail) {
            List.cons(head, tail).append(u) = List.cons(head, tail.append(u))
            List.cons(head, tail).append(u).length = List.cons(head, tail.append(u)).length
            List.cons(head, tail.append(u)).length = tail.append(u).length + Nat.1
            tail.append(u).length = tail.length + Nat.1
            List.cons(head, tail).append(u).length = tail.length + Nat.1 + Nat.1
            List.cons(head, tail).length = tail.length + Nat.1
            List.cons(head, tail).append(u).length = List.cons(head, tail).length + Nat.1
            p(List.cons(head, tail))
        }
    }
    forall(head: T, tail: List[T]) { p(tail) implies p(List.cons(head, tail)) }
    function[T0](x0: List[T0] -> Bool) {
        not forall(x1: T0, x2: List[T0]) { not x0(x2) or x0(List.cons(x1, x2)) } or
        not x0(List.nil[T0]) or forall(x3: List[T0]) { x0(x3) } = true
    }[T](p)
    function[T0](x0: List[T0] -> Bool, x1: List[T0]) {
        not forall(x2: T0, x3: List[T0]) { not x0(x3) or x0(List.cons(x2, x3)) } or
        not x0(List.nil[T0]) or x0(x1)
    }[T](p, xs)
    p(xs)
    if xs.append(u).length = xs.length + Nat.1 {
    }
}

/// The greedy extension never shortens the walk.
theorem greedy_path_length_ge[V](g: SimpleGraph[V], s: FiniteSet[V], a: V, steps: List[V], fuel: Nat) {
    steps.length <= greedy_path(g, s, a, steps, fuel).length
} by {
    define p(f: Nat) -> Bool {
        forall(a2: V, steps2: List[V]) {
            steps2.length <= greedy_path(g, s, a2, steps2, f).length
        }
    }
    forall(a2: V, steps2: List[V]) {
        greedy_path(g, s, a2, steps2, Nat.0) = steps2
        steps2.length <= greedy_path(g, s, a2, steps2, Nat.0).length
    }
    p(Nat.0)
    forall(f: Nat) {
        if p(f) {
            forall(a2: V, steps2: List[V]) {
                if exists(u: V) { greedy_guard(g, s, a2, steps2, u) } {
                    greedy_path(g, s, a2, steps2, f.suc) =
                        greedy_path(g, s, a2, steps2.append(greedy_chosen(g, s, a2, steps2)), f)
                    p(f) = forall(a3: V, steps3: List[V]) {
                        steps3.length <= greedy_path(g, s, a3, steps3, f).length
                    }
                    steps2.append(greedy_chosen(g, s, a2, steps2)).length <= greedy_path(g, s, a2, steps2.append(greedy_chosen(g, s, a2, steps2)), f).length
                    append_length_one(steps2, greedy_chosen(g, s, a2, steps2))
                    steps2.append(greedy_chosen(g, s, a2, steps2)).length = steps2.length + Nat.1
                    steps2.length <= steps2.append(greedy_chosen(g, s, a2, steps2)).length
                    lte_trans(steps2.length, steps2.append(greedy_chosen(g, s, a2, steps2)).length, greedy_path(g, s, a2, steps2, f.suc).length)
                    steps2.length <= greedy_path(g, s, a2, steps2, f.suc).length
                } else {
                    greedy_path(g, s, a2, steps2, f.suc) = steps2
                    steps2.length <= greedy_path(g, s, a2, steps2, f.suc).length
                }
                steps2.length <= greedy_path(g, s, a2, steps2, f.suc).length
            }
            p(f.suc)
        }
    }
    forall(f: Nat) { p(f) implies p(f.suc) }
    p(Nat.0) and forall(f: Nat) { p(f) implies p(f.suc) }
    alt_induction(p)
    forall(f: Nat) { p(f) }
    p(fuel)
    p(fuel) = forall(a3: V, steps3: List[V]) {
        steps3.length <= greedy_path(g, s, a3, steps3, fuel).length
    }
    steps.length <= greedy_path(g, s, a, steps, fuel).length
}

/// A unique list of elements of a finite set is no longer than the set.
theorem unique_list_le_card[T](items: List[T], s: FiniteSet[T]) {
    items.is_unique and (forall(x: T) { items.contains(x) implies s.contains(x) })
        implies items.length <= fs_card(s)
} by {
    if items.is_unique and (forall(x: T) { items.contains(x) implies s.contains(x) }) {
        forall(x: T) {
            if fs_from_list(items).contains(x) {
                fs_from_list_contains_eq(items, x)
                fs_from_list(items).contains(x) = items.contains(x)
                items.contains(x)
                items.contains(x) implies s.contains(x)
                s.contains(x)
            }
            fs_from_list(items).contains(x) implies s.contains(x)
        }
        fs_subset_eq_intro(fs_from_list(items), s)
        fs_from_list(items).subset_eq(s)
        fs_card_mono(fs_from_list(items), s)
        fs_card(fs_from_list(items)) <= fs_card(s)
        finite_set_from_unique_list_cardinality_is_length(items)
        fs_from_list(items).cardinality_is(items.length)
        fs_card_eq_of_cardinality_is(fs_from_list(items), items.length)
        fs_card(fs_from_list(items)) = items.length
        items.length <= fs_card(s)
    }
}

/// A path in a set visits no more vertices than the set has.
theorem path_vertices_le_card[V](g: SimpleGraph[V], s: FiniteSet[V], a: V, steps: List[V]) {
    simple_graph_path_in(g, s, a, steps) implies
        simple_graph_walk_vertices(a, steps).length <= fs_card(s)
} by {
    if simple_graph_path_in(g, s, a, steps) {
        path_in_vertices_unique(g, s, a, steps)
        simple_graph_walk_vertices(a, steps).is_unique
        path_in_start_in(g, s, a, steps)
        s.contains(a)
        simple_graph_walk_vertices(a, steps) = List.cons(a, steps)
        forall(x: V) {
            if List.cons(a, steps).contains(x) {
                List.cons(a, steps).contains(x) = (x = a or steps.contains(x))
                if x = a {
                    s.contains(x)
                }
                if steps.contains(x) {
                    path_in_steps_in(g, s, a, steps, x)
                    s.contains(x)
                }
                s.contains(x)
            }
            List.cons(a, steps).contains(x) implies s.contains(x)
        }
        unique_list_le_card(List.cons(a, steps), s)
        List.cons(a, steps).is_unique and (forall(x: V) { List.cons(a, steps).contains(x) implies s.contains(x) }) implies List.cons(a, steps).length <= fs_card(s)
        List.cons(a, steps).is_unique
        forall(x: V) { List.cons(a, steps).contains(x) implies s.contains(x) }
        List.cons(a, steps).is_unique and (forall(x: V) { List.cons(a, steps).contains(x) implies s.contains(x) })
        List.cons(a, steps).length <= fs_card(s)
        simple_graph_walk_vertices(a, steps).length <= fs_card(s)
    }
}

/// A walk is stuck when every neighbor of its endpoint is already on it.
define greedy_stuck[V](g: SimpleGraph[V], s: FiniteSet[V], a: V, steps: List[V]) -> Bool {
    forall(u: V) {
        s.contains(u) and g.adj(walk_last(a, steps), u) implies (u = a or steps.contains(u))
    }
}

/// The greedy walk either gets stuck or spends its whole fuel budget.
theorem greedy_path_stuck_or_exhausted[V](g: SimpleGraph[V], s: FiniteSet[V], a: V, steps: List[V], fuel: Nat) {
    greedy_stuck(g, s, a, greedy_path(g, s, a, steps, fuel)) or
    (greedy_path(g, s, a, steps, fuel).length = steps.length + fuel)
} by {
    define p(f: Nat) -> Bool {
        forall(a2: V, steps2: List[V]) {
            greedy_stuck(g, s, a2, greedy_path(g, s, a2, steps2, f)) or
            (greedy_path(g, s, a2, steps2, f).length = steps2.length + f)
        }
    }
    forall(a2: V, steps2: List[V]) {
        greedy_path(g, s, a2, steps2, Nat.0) = steps2
        greedy_path(g, s, a2, steps2, Nat.0).length = steps2.length + Nat.0
        greedy_stuck(g, s, a2, greedy_path(g, s, a2, steps2, Nat.0)) or
            (greedy_path(g, s, a2, steps2, Nat.0).length = steps2.length + Nat.0)
    }
    p(Nat.0)
    forall(f: Nat) {
        if p(f) {
            forall(a2: V, steps2: List[V]) {
                if exists(u: V) { greedy_guard(g, s, a2, steps2, u) } {
                    greedy_path(g, s, a2, steps2, f.suc) =
                        greedy_path(g, s, a2, steps2.append(greedy_chosen(g, s, a2, steps2)), f)
                    p(f) = forall(a3: V, steps3: List[V]) {
                        greedy_stuck(g, s, a3, greedy_path(g, s, a3, steps3, f)) or
                        (greedy_path(g, s, a3, steps3, f).length = steps3.length + f)
                    }
                    p(f)
                    greedy_stuck(g, s, a2, greedy_path(g, s, a2, steps2.append(greedy_chosen(g, s, a2, steps2)), f)) or
                        (greedy_path(g, s, a2, steps2.append(greedy_chosen(g, s, a2, steps2)), f).length = steps2.append(greedy_chosen(g, s, a2, steps2)).length + f)
                    if greedy_stuck(g, s, a2, greedy_path(g, s, a2, steps2.append(greedy_chosen(g, s, a2, steps2)), f)) {
                        greedy_path(g, s, a2, steps2, f.suc) =
                            greedy_path(g, s, a2, steps2.append(greedy_chosen(g, s, a2, steps2)), f)
                        greedy_stuck(g, s, a2, greedy_path(g, s, a2, steps2, f.suc)) or
                            (greedy_path(g, s, a2, steps2, f.suc).length = steps2.length + f.suc)
                    }
                    if not greedy_stuck(g, s, a2, greedy_path(g, s, a2, steps2.append(greedy_chosen(g, s, a2, steps2)), f)) {
                        greedy_path(g, s, a2, steps2.append(greedy_chosen(g, s, a2, steps2)), f).length = steps2.append(greedy_chosen(g, s, a2, steps2)).length + f
                        append_length_one(steps2, greedy_chosen(g, s, a2, steps2))
                        steps2.append(greedy_chosen(g, s, a2, steps2)).length = steps2.length + Nat.1
                        greedy_path(g, s, a2, steps2, f.suc).length = greedy_path(g, s, a2, steps2.append(greedy_chosen(g, s, a2, steps2)), f).length
                        greedy_path(g, s, a2, steps2, f.suc).length = steps2.append(greedy_chosen(g, s, a2, steps2)).length + f
                        greedy_path(g, s, a2, steps2, f.suc).length = steps2.length + Nat.1 + f
                        steps2.length + Nat.1 + f = steps2.length + f.suc
                        greedy_path(g, s, a2, steps2, f.suc).length = steps2.length + f.suc
                        greedy_stuck(g, s, a2, greedy_path(g, s, a2, steps2, f.suc)) or
                            (greedy_path(g, s, a2, steps2, f.suc).length = steps2.length + f.suc)
                    }
                    greedy_stuck(g, s, a2, greedy_path(g, s, a2, steps2, f.suc)) or
                        (greedy_path(g, s, a2, steps2, f.suc).length = steps2.length + f.suc)
                } else {
                    not exists(u: V) { greedy_guard(g, s, a2, steps2, u) }
                    greedy_path(g, s, a2, steps2, f.suc) = steps2
                    forall(u: V) {
                        if s.contains(u) and g.adj(walk_last(a2, steps2), u) {
                            if u = a2 {
                                u = a2 or steps2.contains(u)
                            }
                            if not u = a2 {
                                u != a2
                                if steps2.contains(u) {
                                    u = a2 or steps2.contains(u)
                                }
                                if not steps2.contains(u) {
                                    greedy_guard(g, s, a2, steps2, u) =
                                        (s.contains(u) and g.adj(walk_last(a2, steps2), u) and u != a2 and not steps2.contains(u))
                                    greedy_guard(g, s, a2, steps2, u)
                                    exists(u2: V) { greedy_guard(g, s, a2, steps2, u2) }
                                    not exists(u2: V) { greedy_guard(g, s, a2, steps2, u2) }
                                    false
                                }
                                u = a2 or steps2.contains(u)
                            }
                            u = a2 or steps2.contains(u)
                        }
                        s.contains(u) and g.adj(walk_last(a2, steps2), u) implies (u = a2 or steps2.contains(u))
                    }
                    forall(u: V) {
                        s.contains(u) and g.adj(walk_last(a2, steps2), u) implies (u = a2 or steps2.contains(u))
                    }
                    greedy_stuck(g, s, a2, steps2) =
                        forall(u: V) {
                            s.contains(u) and g.adj(walk_last(a2, steps2), u) implies (u = a2 or steps2.contains(u))
                        }
                    greedy_stuck(g, s, a2, steps2)
                    greedy_path(g, s, a2, steps2, f.suc) = steps2
                    greedy_stuck(g, s, a2, greedy_path(g, s, a2, steps2, f.suc)) = greedy_stuck(g, s, a2, steps2)
                    greedy_stuck(g, s, a2, greedy_path(g, s, a2, steps2, f.suc))
                    greedy_stuck(g, s, a2, greedy_path(g, s, a2, steps2, f.suc)) or
                        (greedy_path(g, s, a2, steps2, f.suc).length = steps2.length + f.suc)
                }
                greedy_stuck(g, s, a2, greedy_path(g, s, a2, steps2, f.suc)) or
                    (greedy_path(g, s, a2, steps2, f.suc).length = steps2.length + f.suc)
            }
            p(f.suc)
        }
    }
    forall(f: Nat) { p(f) implies p(f.suc) }
    p(Nat.0) and forall(f: Nat) { p(f) implies p(f.suc) }
    alt_induction(p)
    forall(f: Nat) { p(f) }
    p(fuel)
    p(fuel) = forall(a3: V, steps3: List[V]) {
        greedy_stuck(g, s, a3, greedy_path(g, s, a3, steps3, fuel)) or
        (greedy_path(g, s, a3, steps3, fuel).length = steps3.length + fuel)
    }
    greedy_stuck(g, s, a, greedy_path(g, s, a, steps, fuel)) or
        (greedy_path(g, s, a, steps, fuel).length = steps.length + fuel)
}

/// A greedy walk with more fuel than the set has vertices gets stuck.
theorem greedy_path_maximal[V](g: SimpleGraph[V], s: FiniteSet[V], a: V, steps: List[V], fuel: Nat) {
    simple_graph_path_in(g, s, a, steps) and fs_card(s) < steps.length + fuel
        implies greedy_stuck(g, s, a, greedy_path(g, s, a, steps, fuel))
} by {
    if simple_graph_path_in(g, s, a, steps) and fs_card(s) < steps.length + fuel {
        greedy_path_stuck_or_exhausted(g, s, a, steps, fuel)
        greedy_stuck(g, s, a, greedy_path(g, s, a, steps, fuel)) or
            (greedy_path(g, s, a, steps, fuel).length = steps.length + fuel)
        if greedy_path(g, s, a, steps, fuel).length = steps.length + fuel {
            greedy_path_preserves_path(g, s, a, steps, fuel)
            simple_graph_path_in(g, s, a, greedy_path(g, s, a, steps, fuel))
            path_vertices_le_card(g, s, a, greedy_path(g, s, a, steps, fuel))
            simple_graph_walk_vertices(a, greedy_path(g, s, a, steps, fuel)).length <= fs_card(s)
            simple_graph_walk_vertices(a, greedy_path(g, s, a, steps, fuel)) = List.cons(a, greedy_path(g, s, a, steps, fuel))
            List.cons(a, greedy_path(g, s, a, steps, fuel)).length <= fs_card(s)
            List.cons(a, greedy_path(g, s, a, steps, fuel)).length = greedy_path(g, s, a, steps, fuel).length + Nat.1
            greedy_path(g, s, a, steps, fuel).length + Nat.1 <= fs_card(s)
            greedy_path(g, s, a, steps, fuel).length + Nat.1 = steps.length + fuel + Nat.1
            steps.length + fuel + Nat.1 <= fs_card(s)
            fs_card(s) < steps.length + fuel
            lt_suc(steps.length + fuel)
            steps.length + fuel < (steps.length + fuel).suc
            steps.length + fuel < Nat.1 + (steps.length + fuel)
            lt_trans(fs_card(s), steps.length + fuel, Nat.1 + (steps.length + fuel))
            fs_card(s) < Nat.1 + (steps.length + fuel)
            lte_and_lt(Nat.1 + (steps.length + fuel), fs_card(s), Nat.1 + (steps.length + fuel))
            Nat.1 + (steps.length + fuel) < Nat.1 + (steps.length + fuel)
            false
        }
        not (greedy_path(g, s, a, steps, fuel).length = steps.length + fuel)
        greedy_stuck(g, s, a, greedy_path(g, s, a, steps, fuel))
    }
}

/// A list of length zero is the empty list.
theorem length_zero_imp_nil[T](xs: List[T]) {
    xs.length = Nat.0 implies xs = List.nil[T]
} by {
    if xs.length = Nat.0 {
        match xs {
            List.nil[T] {
                xs = List.nil[T]
            }
            List.cons(head, tail) {
                List.cons(head, tail).length = tail.length + Nat.1
                xs.length = tail.length + Nat.1
                tail.length + Nat.1 = Nat.0
                tail.length + Nat.1 = tail.length.suc
                tail.length.suc = Nat.0
                false
            }
        }
        xs = List.nil[T]
    }
}

/// The first occurrence index of a member lies inside the list.
theorem find_first_idx_contains[T](xs: List[T], item: T) {
    xs.contains(item) implies xs.find_first_idx(item) < xs.length
} by {
    define p(ys: List[T]) -> Bool {
        ys.contains(item) implies ys.find_first_idx(item) < ys.length
    }
    if List.nil[T].contains(item) {
        List.nil[T].contains(item) = false
        false
    }
    p(List.nil[T])
    forall(head: T, tail: List[T]) {
        if p(tail) {
            if List.cons(head, tail).contains(item) {
                if head = item {
                    List.cons(head, tail).find_first_idx(item) = Nat.0
                    List.cons(head, tail).length = tail.length + Nat.1
                    Nat.0 < List.cons(head, tail).length
                    List.cons(head, tail).find_first_idx(item) < List.cons(head, tail).length
                }
                if not head = item {
                    head != item
                    find_first_idx_cons(head, tail, item)
                    List.cons(head, tail).find_first_idx(item) = Nat.1 + tail.find_first_idx(item)
                    List.cons(head, tail).contains(item) = (item = head or tail.contains(item))
                    if item = head {
                        head = item
                        not head = item
                        false
                    }
                    tail.contains(item)
                    p(tail)
                    tail.find_first_idx(item) < tail.length
                    Nat.1 + tail.find_first_idx(item) < Nat.1 + tail.length
                    Nat.1 + tail.length = tail.length.suc
                    Nat.1 + tail.find_first_idx(item) < tail.length.suc
                    List.cons(head, tail).length = tail.length + Nat.1
                    List.cons(head, tail).length = tail.length.suc
                    Nat.1 + tail.find_first_idx(item) < List.cons(head, tail).length
                    List.cons(head, tail).find_first_idx(item) < List.cons(head, tail).length
                }
                List.cons(head, tail).find_first_idx(item) < List.cons(head, tail).length
            }
            p(List.cons(head, tail))
        }
    }
    forall(head: T, tail: List[T]) { p(tail) implies p(List.cons(head, tail)) }
    function[T0](x0: List[T0] -> Bool) {
        not forall(x1: T0, x2: List[T0]) { not x0(x2) or x0(List.cons(x1, x2)) } or
        not x0(List.nil[T0]) or forall(x3: List[T0]) { x0(x3) } = true
    }[T](p)
    function[T0](x0: List[T0] -> Bool, x1: List[T0]) {
        not forall(x2: T0, x3: List[T0]) { not x0(x3) or x0(List.cons(x2, x3)) } or
        not x0(List.nil[T0]) or x0(x1)
    }[T](p, xs)
    p(xs)
    if xs.contains(item) {
        xs.find_first_idx(item) < xs.length
    }
}

/// The penultimate vertex sits at the second-to-last index.
theorem penultimate_idx[V](start: V, steps: List[V]) {
    Nat.2 <= steps.length implies
        steps.get_idx(steps.length - Nat.2) = Option.some(walk_penultimate(start, steps))
} by {
    define p(xs: List[V]) -> Bool {
        forall(x: V) {
            not (Nat.2 <= xs.length) or
            (xs.get_idx(xs.length - Nat.2) = Option.some(walk_penultimate(x, xs)))
        }
    }
    forall(x: V) {
        if Nat.2 <= List.nil[V].length {
            List.nil[V].length = Nat.0
            Nat.2 <= Nat.0
            only_zero_lte_zero(Nat.2)
            Nat.2 = Nat.0
            false
        }
        not (Nat.2 <= List.nil[V].length)
        not (Nat.2 <= List.nil[V].length) or
            (List.nil[V].get_idx(List.nil[V].length - Nat.2) = Option.some(walk_penultimate(x, List.nil[V])))
        p(List.nil[V])
    }
    p(List.nil[V])
    forall(head: V, tail: List[V]) {
        if p(tail) {
            forall(x: V) {
                if Nat.2 <= List.cons(head, tail).length {
                    List.cons(head, tail).length = tail.length + Nat.1
                    if tail.length = Nat.0 {
                        tail = List.nil[V]
                        List.cons(head, List.nil[V]).length = List.nil[V].length + Nat.1
                        List.nil[V].length = Nat.0
                        List.cons(head, List.nil[V]).length = Nat.1
                        Nat.2 <= Nat.1
                        lt_suc(Nat.1)
                        Nat.1 < Nat.2
                        lte_and_lt(Nat.2, Nat.1, Nat.2)
                        Nat.2 < Nat.2
                        false
                    }
                    if not tail.length = Nat.0 {
                        match tail {
                            List.nil[V] {
                                false
                            }
                            List.cons(head2, tail2) {
                                tail = List.cons(head2, tail2)
                                List.cons(head2, tail2).length = tail2.length + Nat.1
                                tail.length = tail2.length + Nat.1
                                List.cons(head, tail).length = tail.length + Nat.1
                                if tail2.length = Nat.0 {
                                    tail2 = List.nil[V]
                                    List.cons(head, List.cons(head2, List.nil[V])).length =
                                        List.cons(head2, List.nil[V]).length + Nat.1
                                    List.cons(head2, List.nil[V]).length = List.nil[V].length + Nat.1
                                    List.nil[V].length = Nat.0
                                    List.cons(head2, List.nil[V]).length = Nat.1
                                    List.cons(head, List.cons(head2, List.nil[V])).length = Nat.2
                                    List.cons(head, List.cons(head2, List.nil[V])).length - Nat.2 = Nat.0
                                    List.cons(head, List.cons(head2, List.nil[V])).get_idx(Nat.0) = Option.some(head)
                                    walk_penultimate(x, List.cons(head, List.cons(head2, List.nil[V]))) = head
                                    List.cons(head, List.cons(head2, List.nil[V])).get_idx(List.cons(head, List.cons(head2, List.nil[V])).length - Nat.2) = Option.some(walk_penultimate(x, List.cons(head, List.cons(head2, List.nil[V]))))
                                    List.cons(head, tail).get_idx(List.cons(head, tail).length - Nat.2) = Option.some(walk_penultimate(x, List.cons(head, tail)))
                                }
                                if not tail2.length = Nat.0 {
                                    p(tail) = forall(x2: V) {
                                        not (Nat.2 <= tail.length) or
                                        (tail.get_idx(tail.length - Nat.2) = Option.some(walk_penultimate(x2, tail)))
                                    }
                                    Nat.1 <= tail2.length
                                    tail.length = tail2.length + Nat.1
                                    Nat.2 <= tail2.length + Nat.1
                                    Nat.2 <= tail.length
                                    p(tail)
                                    tail.get_idx(tail.length - Nat.2) = Option.some(walk_penultimate(head, tail))
                                    List.cons(head, tail).length - Nat.2 = tail.length + Nat.1 - Nat.2
                                    Nat.1 <= Nat.2
                                    lte_trans(Nat.1, Nat.2, tail.length)
                                    Nat.1 <= tail.length
                                    sub_suc_suc(tail.length, Nat.1)
                                    (tail.length + Nat.1) - (Nat.1 + Nat.1) = tail.length - Nat.1
                                    Nat.1 + Nat.1 = Nat.2
                                    (tail.length + Nat.1) - Nat.2 = tail.length - Nat.1
                                    tail.length + Nat.1 - Nat.2 = tail.length - Nat.1
                                    List.cons(head, tail).length - Nat.2 = tail.length - Nat.1
                                    get_idx_cons(head, tail, tail.length - Nat.2)
                                    List.cons(head, tail).get_idx(tail.length - Nat.2 + Nat.1) = tail.get_idx(tail.length - Nat.2)
                                    add_sub(tail.length, Nat.2)
                                    tail.length - Nat.2 + Nat.2 = tail.length
                                    tail.length - Nat.2 + Nat.1 + Nat.1 = tail.length
                                    add_sub(tail.length, Nat.1)
                                    tail.length - Nat.1 + Nat.1 = tail.length
                                    add_cancels_right(tail.length - Nat.2 + Nat.1, tail.length - Nat.1, Nat.1)
                                    tail.length - Nat.2 + Nat.1 = tail.length - Nat.1
                                    List.cons(head, tail).get_idx(tail.length - Nat.1) = tail.get_idx(tail.length - Nat.2)
                                    List.cons(head, tail).get_idx(List.cons(head, tail).length - Nat.2) = Option.some(walk_penultimate(head, tail))
                                    walk_penultimate(x, List.cons(head, List.cons(head2, tail2))) = walk_penultimate(head, List.cons(head2, tail2))
                                    walk_penultimate(x, List.cons(head, tail)) = walk_penultimate(head, tail)
                                    List.cons(head, tail).get_idx(List.cons(head, tail).length - Nat.2) = Option.some(walk_penultimate(x, List.cons(head, tail)))
                                }
                                List.cons(head, tail).get_idx(List.cons(head, tail).length - Nat.2) = Option.some(walk_penultimate(x, List.cons(head, tail)))
                            }
                        }
                        List.cons(head, tail).get_idx(List.cons(head, tail).length - Nat.2) = Option.some(walk_penultimate(x, List.cons(head, tail)))
                    }
                    List.cons(head, tail).get_idx(List.cons(head, tail).length - Nat.2) = Option.some(walk_penultimate(x, List.cons(head, tail)))
                }
                not (Nat.2 <= List.cons(head, tail).length) or
                    (List.cons(head, tail).get_idx(List.cons(head, tail).length - Nat.2) = Option.some(walk_penultimate(x, List.cons(head, tail))))
            }
            p(List.cons(head, tail))
        }
    }
    forall(head: V, tail: List[V]) { p(tail) implies p(List.cons(head, tail)) }
    function[T0](x0: List[T0] -> Bool) {
        not forall(x1: T0, x2: List[T0]) { not x0(x2) or x0(List.cons(x1, x2)) } or
        not x0(List.nil[T0]) or forall(x3: List[T0]) { x0(x3) } = true
    }[V](p)
    function[T0](x0: List[T0] -> Bool, x1: List[T0]) {
        not forall(x2: T0, x3: List[T0]) { not x0(x3) or x0(List.cons(x2, x3)) } or
        not x0(List.nil[T0]) or x0(x1)
    }[V](p, steps)
    p(steps)
    p(steps) = forall(x2: V) {
        not (Nat.2 <= steps.length) or
        (steps.get_idx(steps.length - Nat.2) = Option.some(walk_penultimate(x2, steps)))
    }
    if Nat.2 <= steps.length {
        if not (Nat.2 <= steps.length) {
            false
        }
        steps.get_idx(steps.length - Nat.2) = Option.some(walk_penultimate(start, steps))
    }
    Nat.2 <= steps.length implies steps.get_idx(steps.length - Nat.2) = Option.some(walk_penultimate(start, steps))
}

/// A bound by a successor that is not the successor itself is a bound by the number.
theorem lte_suc_ne_imp_lte(a: Nat, b: Nat) {
    a <= b + Nat.1 and a != b + Nat.1 implies a <= b
} by {
    if a <= b + Nat.1 and a != b + Nat.1 {
        lt_suc(b + Nat.1)
        b + Nat.1 < (b + Nat.1).suc
        lte_and_lt(a, b + Nat.1, (b + Nat.1).suc)
        a < (b + Nat.1).suc
        lt_suc_right(a, b + Nat.1)
        a = b + Nat.1 or a < b + Nat.1
        if a = b + Nat.1 {
            a != b + Nat.1
            false
        }
        not (a = b + Nat.1)
        a < b + Nat.1
        lt_imp_lte_suc(a, b + Nat.1)
        a.suc <= b + Nat.1
        b + Nat.1 = b.suc
        a.suc <= b.suc
        lte_cancel_suc(a, b)
        a <= b
    }
}

/// A singleton list has length one.
theorem singleton_length[T](u: T) {
    List.singleton(u).length = Nat.1
} by {
    List.cons(u, List.nil[T]).length = List.nil[T].length + Nat.1
    List.nil[T].length = Nat.0
    List.cons(u, List.nil[T]).length = Nat.1
    List.singleton(u).length = Nat.1
}

/// The endpoint of a one-step stuck acyclic path is a leaf with the start as witness.
theorem path_endpoint_leaf_one[V](g: SimpleGraph[V], s: FiniteSet[V], a: V, r: List[V]) {
    simple_graph_path_in(g, s, a, r) and r.length = Nat.1 and greedy_stuck(g, s, a, r)
        implies exists(v: V) { s.contains(v) and is_leaf(g, s, v) }
} by {
    if simple_graph_path_in(g, s, a, r) and r.length = Nat.1 and greedy_stuck(g, s, a, r) {
        path_in_walk(g, s, a, r)
        simple_graph_walk(g, a, walk_last(a, r), r)
        walk_penultimate_adj(g, a, walk_last(a, r), r)
        Nat.1 <= Nat.2
        lte_trans(Nat.1, Nat.2, r.length)
        Nat.1 <= r.length
        g.adj(walk_penultimate(a, r), walk_last(a, r))
        match r {
            List.nil[V] {
                List.nil[V].length = Nat.0
                r.length = Nat.0
                r.length = Nat.1
                Nat.1 = Nat.0
                false
            }
            List.cons(head, tail2) {
                match tail2 {
                    List.nil[V] {
                        tail2 = List.nil[V]
                        r = List.cons(head, List.nil[V])
                        walk_penultimate(a, List.cons(head, List.nil[V])) = a
                        walk_penultimate(a, r) = a
                        g.adj(a, walk_last(a, r))
                        s.contains(a)
                        forall(u: V) {
                            if s.contains(u) and g.adj(walk_last(a, r), u) {
                                greedy_stuck(g, s, a, r) =
                                    forall(u2: V) {
                                        s.contains(u2) and g.adj(walk_last(a, r), u2) implies (u2 = a or r.contains(u2))
                                    }
                                greedy_stuck(g, s, a, r)
                                u = a or r.contains(u)
                                if u = a {
                                    u = a
                                }
                                if not u = a {
                                    u != a
                                    r.contains(u)
                                    r.contains(u) = (u = head or List.nil[V].contains(u))
                                    u = head
                                    walk_last(a, r) = walk_last(a, List.cons(head, List.nil[V]))
                                    walk_last(a, List.cons(head, List.nil[V])) = walk_last(head, List.nil[V])
                                    walk_last(head, List.nil[V]) = head
                                    walk_last(a, r) = head
                                    u = walk_last(a, r)
                                    g.adj(walk_last(a, r), u)
                                    g.adj(u, u)
                                    simple_graph_adj_ne(g, u, u)
                                    g.adj(u, u) implies u != u
                                    u != u
                                    false
                                }
                                u = a
                            }
                            s.contains(u) and g.adj(walk_last(a, r), u) implies u = a
                        }
                        forall(u: V) {
                            s.contains(u) and g.adj(walk_last(a, r), u) implies u = a
                        }
                        s.contains(walk_last(a, r))
                        path_in_endpoint_in(g, s, a, r)
                        s.contains(a)
                        g.adj(walk_last(a, r), a)
                        forall(w: V) { s.contains(w) and g.adj(walk_last(a, r), w) implies w = a }
                        s.contains(a) and g.adj(walk_last(a, r), a) and
                            (forall(w: V) { s.contains(w) and g.adj(walk_last(a, r), w) implies w = a })
                        is_leaf(g, s, walk_last(a, r)) = exists(u: V) {
                            s.contains(u) and g.adj(walk_last(a, r), u) and
                                forall(w: V) {
                                    s.contains(w) and g.adj(walk_last(a, r), w) implies w = u
                                }
                        }
                        is_leaf(g, s, walk_last(a, r))
                        s.contains(walk_last(a, r))
                        exists(v: V) { s.contains(v) and is_leaf(g, s, v) }
                    }
                    List.cons(head2, tail3) {
                        List.cons(head, List.cons(head2, tail3)).length = List.cons(head2, tail3).length + Nat.1
                        r.length = List.cons(head2, tail3).length + Nat.1
                        r.length = Nat.1
                        List.cons(head2, tail3).length + Nat.1 = Nat.1
                        List.cons(head2, tail3).length = Nat.0
                        List.cons(head2, tail3).length = tail3.length + Nat.1
                        tail3.length + Nat.1 = Nat.0
                        false
                    }
                }
                exists(v: V) { s.contains(v) and is_leaf(g, s, v) }
            }
        }
        exists(v: V) { s.contains(v) and is_leaf(g, s, v) }
    }
}

/// In a list without repeats, the first occurrence is the only occurrence.
theorem unique_not_contains_drop_suc[T](xs: List[T], w: T) {
    xs.is_unique and xs.contains(w) implies
        not xs.drop(xs.find_first_idx(w) + Nat.1).contains(w)
} by {
    define p(ys: List[T]) -> Bool {
        ys.is_unique and ys.contains(w) implies
            not ys.drop(ys.find_first_idx(w) + Nat.1).contains(w)
    }
    if List.nil[T].is_unique and List.nil[T].contains(w) {
        List.nil[T].contains(w) = false
        false
    }
    p(List.nil[T])
    forall(head: T, tail: List[T]) {
        if p(tail) {
            if List.cons(head, tail).is_unique and List.cons(head, tail).contains(w) {
                if head = w {
                    List.cons(head, tail).find_first_idx(w) = Nat.0
                    List.cons(head, tail).drop(Nat.0 + Nat.1) = tail.drop(Nat.0)
                    List.cons(head, tail).drop(Nat.1) = tail
                    unique_cons_head_not_in_tail(head, tail)
                    not tail.contains(head)
                    not tail.contains(w)
                    List.cons(head, tail).drop(List.cons(head, tail).find_first_idx(w) + Nat.1) = tail
                    not List.cons(head, tail).drop(List.cons(head, tail).find_first_idx(w) + Nat.1).contains(w)
                }
                if not head = w {
                    head != w
                    find_first_idx_cons(head, tail, w)
                    List.cons(head, tail).find_first_idx(w) = Nat.1 + tail.find_first_idx(w)
                    List.cons(head, tail).contains(w) = (w = head or tail.contains(w))
                    if w = head {
                        head = w
                        not head = w
                        false
                    }
                    tail.contains(w)
                    unique_implies_tail_unique(head, tail)
                    tail.is_unique
                    p(tail)
                    not tail.drop(tail.find_first_idx(w) + Nat.1).contains(w)
                    drop_cons(head, tail, tail.find_first_idx(w) + Nat.1)
                    List.cons(head, tail).drop(tail.find_first_idx(w) + Nat.1 + Nat.1) =
                        tail.drop(tail.find_first_idx(w) + Nat.1)
                    List.cons(head, tail).drop(Nat.1 + tail.find_first_idx(w) + Nat.1) =
                        tail.drop(tail.find_first_idx(w) + Nat.1)
                    List.cons(head, tail).drop(List.cons(head, tail).find_first_idx(w) + Nat.1) =
                        tail.drop(tail.find_first_idx(w) + Nat.1)
                    not List.cons(head, tail).drop(List.cons(head, tail).find_first_idx(w) + Nat.1).contains(w)
                }
                not List.cons(head, tail).drop(List.cons(head, tail).find_first_idx(w) + Nat.1).contains(w)
            }
            p(List.cons(head, tail))
        }
    }
    forall(head: T, tail: List[T]) { p(tail) implies p(List.cons(head, tail)) }
    function[T0](x0: List[T0] -> Bool) {
        not forall(x1: T0, x2: List[T0]) { not x0(x2) or x0(List.cons(x1, x2)) } or
        not x0(List.nil[T0]) or forall(x3: List[T0]) { x0(x3) } = true
    }[T](p)
    function[T0](x0: List[T0] -> Bool, x1: List[T0]) {
        not forall(x2: T0, x3: List[T0]) { not x0(x3) or x0(List.cons(x2, x3)) } or
        not x0(List.nil[T0]) or x0(x1)
    }[T](p, xs)
    p(xs)
    if xs.is_unique and xs.contains(w) {
        not xs.drop(xs.find_first_idx(w) + Nat.1).contains(w)
    }
}

/// Adding a common amount preserves strict inequality on the right.
theorem lt_add_cancel_left(a: Nat, b: Nat, c: Nat) {
    a + b < a + c implies b < c
} by {
    define p(aa: Nat) -> Bool {
        forall(bb: Nat, cc: Nat) { aa + bb < aa + cc implies bb < cc }
    }
    forall(bb: Nat, cc: Nat) {
        if Nat.0 + bb < Nat.0 + cc {
            bb < cc
        }
        p(Nat.0)
    }
    p(Nat.0)
    forall(aa: Nat) {
        if p(aa) {
            forall(bb: Nat, cc: Nat) {
                if aa.suc + bb < aa.suc + cc {
                    add_suc_left(aa, bb)
                    aa.suc + bb = (aa + bb).suc
                    add_suc_left(aa, cc)
                    aa.suc + cc = (aa + cc).suc
                    (aa + bb).suc < (aa + cc).suc
                    lt_cancel_suc(aa + bb, aa + cc)
                    aa + bb < aa + cc
                    p(aa) = forall(b2: Nat, c2: Nat) { aa + b2 < aa + c2 implies b2 < c2 }
                    p(aa)
                    bb < cc
                }
                aa.suc + bb < aa.suc + cc implies bb < cc
            }
            p(aa.suc)
        }
    }
    forall(aa: Nat) { p(aa) implies p(aa.suc) }
    p(Nat.0) and forall(aa: Nat) { p(aa) implies p(aa.suc) }
    alt_induction(p)
    forall(aa: Nat) { p(aa) }
    p(a)
    p(a) = forall(b2: Nat, c2: Nat) { a + b2 < a + c2 implies b2 < c2 }
    if a + b < a + c {
        b < c
    }
}

/// The walk from a member of the target list to the finish is the suffix after its first occurrence.
theorem walk_suffix_at[V](g: SimpleGraph[V], a: V, b: V, steps: List[V], w: V) {
    (simple_graph_walk(g, a, b, steps) and steps.contains(w) implies
        simple_graph_walk(g, w, b, steps.drop(steps.find_first_idx(w) + Nat.1)))
} by {
    define p(ys: List[V]) -> Bool {
        forall(aa: V, bb: V) {
            not (simple_graph_walk(g, aa, bb, ys) and ys.contains(w)) or
            simple_graph_walk(g, w, bb, ys.drop(ys.find_first_idx(w) + Nat.1))
        }
    }
    forall(aa: V, bb: V) {
        if simple_graph_walk(g, aa, bb, List.nil[V]) and List.nil[V].contains(w) {
            List.nil[V].contains(w) = false
            false
        }
        not (simple_graph_walk(g, aa, bb, List.nil[V]) and List.nil[V].contains(w))
        not (simple_graph_walk(g, aa, bb, List.nil[V]) and List.nil[V].contains(w)) or
            simple_graph_walk(g, w, bb, List.nil[V].drop(List.nil[V].find_first_idx(w) + Nat.1))
        p(List.nil[V])
    }
    p(List.nil[V])
    forall(head: V, tail: List[V]) {
        if p(tail) {
            forall(aa: V, bb: V) {
                if simple_graph_walk(g, aa, bb, List.cons(head, tail)) and List.cons(head, tail).contains(w) {
                    simple_graph_walk_cons_iff(g, aa, bb, head, tail)
                    simple_graph_walk(g, aa, bb, List.cons(head, tail)) =
                        (g.adj(aa, head) and simple_graph_walk(g, head, bb, tail))
                    simple_graph_walk(g, head, bb, tail)
                    if head = w {
                        List.cons(head, tail).find_first_idx(w) = Nat.0
                        List.cons(head, tail).drop(Nat.0 + Nat.1) = List.cons(head, tail).drop(Nat.1)
                        List.cons(head, tail).drop(Nat.1) = List.cons(head, tail).tail
                        List.cons(head, tail).tail = tail
                        List.cons(head, tail).drop(Nat.1) = tail
                        List.cons(head, tail).drop(Nat.0 + Nat.1) = tail
                        simple_graph_walk(g, w, bb, tail)
                        simple_graph_walk(g, w, bb, List.cons(head, tail).drop(List.cons(head, tail).find_first_idx(w) + Nat.1))
                        not (simple_graph_walk(g, aa, bb, List.cons(head, tail)) and List.cons(head, tail).contains(w)) or
                            simple_graph_walk(g, w, bb, List.cons(head, tail).drop(List.cons(head, tail).find_first_idx(w) + Nat.1))
                    }
                    if not head = w {
                        head != w
                        List.cons(head, tail).contains(w) = (w = head or tail.contains(w))
                        if w = head {
                            head = w
                            not head = w
                            false
                        }
                        tail.contains(w)
                        p(tail) = forall(aa2: V, bb2: V) {
                            not (simple_graph_walk(g, aa2, bb2, tail) and tail.contains(w)) or
                            simple_graph_walk(g, w, bb2, tail.drop(tail.find_first_idx(w) + Nat.1))
                        }
                        p(tail)
                        not (simple_graph_walk(g, head, bb, tail) and tail.contains(w)) or
                            simple_graph_walk(g, w, bb, tail.drop(tail.find_first_idx(w) + Nat.1))
                        if not (simple_graph_walk(g, head, bb, tail) and tail.contains(w)) {
                            simple_graph_walk(g, head, bb, tail)
                            tail.contains(w)
                            simple_graph_walk(g, head, bb, tail) and tail.contains(w)
                            not (simple_graph_walk(g, head, bb, tail) and tail.contains(w))
                            false
                        }
                        simple_graph_walk(g, w, bb, tail.drop(tail.find_first_idx(w) + Nat.1))
                        find_first_idx_cons(head, tail, w)
                        List.cons(head, tail).find_first_idx(w) = Nat.1 + tail.find_first_idx(w)
                        List.cons(head, tail).find_first_idx(w) + Nat.1 = Nat.1 + tail.find_first_idx(w) + Nat.1
                        drop_cons(head, tail, tail.find_first_idx(w) + Nat.1)
                        List.cons(head, tail).drop(tail.find_first_idx(w) + Nat.1 + Nat.1) =
                            tail.drop(tail.find_first_idx(w) + Nat.1)
                        List.cons(head, tail).drop(Nat.1 + tail.find_first_idx(w) + Nat.1) =
                            tail.drop(tail.find_first_idx(w) + Nat.1)
                        List.cons(head, tail).drop(List.cons(head, tail).find_first_idx(w) + Nat.1) =
                            tail.drop(tail.find_first_idx(w) + Nat.1)
                        simple_graph_walk(g, w, bb, List.cons(head, tail).drop(List.cons(head, tail).find_first_idx(w) + Nat.1))
                        not (simple_graph_walk(g, aa, bb, List.cons(head, tail)) and List.cons(head, tail).contains(w)) or
                            simple_graph_walk(g, w, bb, List.cons(head, tail).drop(List.cons(head, tail).find_first_idx(w) + Nat.1))
                    }
                    not (simple_graph_walk(g, aa, bb, List.cons(head, tail)) and List.cons(head, tail).contains(w)) or
                        simple_graph_walk(g, w, bb, List.cons(head, tail).drop(List.cons(head, tail).find_first_idx(w) + Nat.1))
                }
                if not (simple_graph_walk(g, aa, bb, List.cons(head, tail)) and List.cons(head, tail).contains(w)) {
                    not (simple_graph_walk(g, aa, bb, List.cons(head, tail)) and List.cons(head, tail).contains(w)) or
                        simple_graph_walk(g, w, bb, List.cons(head, tail).drop(List.cons(head, tail).find_first_idx(w) + Nat.1))
                }
                not (simple_graph_walk(g, aa, bb, List.cons(head, tail)) and List.cons(head, tail).contains(w)) or
                    simple_graph_walk(g, w, bb, List.cons(head, tail).drop(List.cons(head, tail).find_first_idx(w) + Nat.1))
            }
            p(List.cons(head, tail))
        }
    }
    forall(head: V, tail: List[V]) { p(tail) implies p(List.cons(head, tail)) }
    function[T0](x0: List[T0] -> Bool) {
        not forall(x1: T0, x2: List[T0]) { not x0(x2) or x0(List.cons(x1, x2)) } or
        not x0(List.nil[T0]) or forall(x3: List[T0]) { x0(x3) } = true
    }[V](p)
    function[T0](x0: List[T0] -> Bool, x1: List[T0]) {
        not forall(x2: T0, x3: List[T0]) { not x0(x3) or x0(List.cons(x2, x3)) } or
        not x0(List.nil[T0]) or x0(x1)
    }[V](p, steps)
    p(steps)
    p(steps) = forall(aa2: V, bb2: V) {
        not (simple_graph_walk(g, aa2, bb2, steps) and steps.contains(w)) or
        simple_graph_walk(g, w, bb2, steps.drop(steps.find_first_idx(w) + Nat.1))
    }
    if simple_graph_walk(g, a, b, steps) and steps.contains(w) {
        if not (simple_graph_walk(g, a, b, steps) and steps.contains(w)) {
            false
        }
        simple_graph_walk(g, w, b, steps.drop(steps.find_first_idx(w) + Nat.1))
    }
    (simple_graph_walk(g, a, b, steps) and steps.contains(w) implies
        simple_graph_walk(g, w, b, steps.drop(steps.find_first_idx(w) + Nat.1)))
}

/// An edge from the endpoint back to the start of a long path makes a cycle.
theorem cycle_of_start_edge[V](g: SimpleGraph[V], s: FiniteSet[V], a: V, r: List[V]) {
    simple_graph_path_in(g, s, a, r) and Nat.2 <= r.length and g.adj(walk_last(a, r), a)
        implies simple_graph_cycle(g, walk_last(a, r), simple_graph_walk_vertices(a, r))
} by {
    if simple_graph_path_in(g, s, a, r) and Nat.2 <= r.length and g.adj(walk_last(a, r), a) {
        path_in_walk(g, s, a, r)
        simple_graph_walk(g, a, walk_last(a, r), r)
        simple_graph_walk_cons_iff(g, walk_last(a, r), walk_last(a, r), a, r)
        simple_graph_walk(g, walk_last(a, r), walk_last(a, r), List.cons(a, r)) =
            (g.adj(walk_last(a, r), a) and simple_graph_walk(g, a, walk_last(a, r), r))
        simple_graph_walk(g, walk_last(a, r), walk_last(a, r), List.cons(a, r))
        simple_graph_walk_vertices(a, r) = List.cons(a, r)
        simple_graph_walk(g, walk_last(a, r), walk_last(a, r), simple_graph_walk_vertices(a, r))
        simple_graph_walk_vertices(a, r).length = List.cons(a, r).length
        List.cons(a, r).length = r.length + Nat.1
        Nat.1 <= Nat.2
        lte_trans(Nat.1, Nat.2, r.length)
        Nat.1 <= r.length
        Nat.1 <= r.length + Nat.1
        Nat.1 <= simple_graph_walk_vertices(a, r).length
        lt_and_lte(Nat.0, Nat.1, simple_graph_walk_vertices(a, r).length)
        Nat.0 < simple_graph_walk_vertices(a, r).length
        simple_graph_closed_walk_intro(g, walk_last(a, r), simple_graph_walk_vertices(a, r))
        simple_graph_closed_walk(g, walk_last(a, r), simple_graph_walk_vertices(a, r))
        Nat.2 <= r.length
        List.cons(a, r).length = r.length + Nat.1
        Nat.3 <= r.length + Nat.1
        Nat.3 <= simple_graph_walk_vertices(a, r).length
        path_in_vertices_unique(g, s, a, r)
        simple_graph_walk_vertices(a, r).is_unique
        simple_graph_cycle(g, walk_last(a, r), simple_graph_walk_vertices(a, r)) =
            (simple_graph_closed_walk(g, walk_last(a, r), simple_graph_walk_vertices(a, r)) and
                Nat.3 <= simple_graph_walk_vertices(a, r).length and
                simple_graph_walk_vertices(a, r).is_unique)
        simple_graph_cycle(g, walk_last(a, r), simple_graph_walk_vertices(a, r))
    }
}

/// An edge from the endpoint to an interior vertex of a long path makes a cycle.
theorem cycle_of_inner_edge[V](g: SimpleGraph[V], s: FiniteSet[V], a: V, r: List[V], w: V, j: Nat) {
    simple_graph_path_in(g, s, a, r) and s.contains(w) and r.contains(w) and
        j = r.find_first_idx(w) and j + Nat.2 < r.length and g.adj(walk_last(a, r), w)
        implies simple_graph_cycle(g, walk_last(a, r), List.cons(w, r.drop(j + Nat.1)))
} by {
    if simple_graph_path_in(g, s, a, r) and s.contains(w) and r.contains(w) and
        j = r.find_first_idx(w) and j + Nat.2 < r.length and g.adj(walk_last(a, r), w) {
        path_in_walk(g, s, a, r)
        simple_graph_walk(g, a, walk_last(a, r), r)
        walk_suffix_at(g, a, walk_last(a, r), r, w)
        (simple_graph_walk(g, a, walk_last(a, r), r) and r.contains(w) implies
            simple_graph_walk(g, w, walk_last(a, r), r.drop(r.find_first_idx(w) + Nat.1)))
        simple_graph_walk(g, w, walk_last(a, r), r.drop(r.find_first_idx(w) + Nat.1))
        j = r.find_first_idx(w)
        r.drop(r.find_first_idx(w) + Nat.1) = r.drop(j + Nat.1)
        simple_graph_walk(g, w, walk_last(a, r), r.drop(j + Nat.1))
        simple_graph_walk_cons_iff(g, walk_last(a, r), walk_last(a, r), w, r.drop(j + Nat.1))
        simple_graph_walk(g, walk_last(a, r), walk_last(a, r), List.cons(w, r.drop(j + Nat.1))) =
            (g.adj(walk_last(a, r), w) and simple_graph_walk(g, w, walk_last(a, r), r.drop(j + Nat.1)))
        simple_graph_walk(g, walk_last(a, r), walk_last(a, r), List.cons(w, r.drop(j + Nat.1)))
        List.cons(w, r.drop(j + Nat.1)).length = r.drop(j + Nat.1).length + Nat.1
        Nat.1 <= r.drop(j + Nat.1).length + Nat.1
        Nat.1 <= List.cons(w, r.drop(j + Nat.1)).length
        Nat.0 < List.cons(w, r.drop(j + Nat.1)).length
        simple_graph_closed_walk_intro(g, walk_last(a, r), List.cons(w, r.drop(j + Nat.1)))
        simple_graph_closed_walk(g, walk_last(a, r), List.cons(w, r.drop(j + Nat.1)))
        drop_length(r, j + Nat.1)
        lt_suc(j + Nat.1)
        j + Nat.1 < j + Nat.2
        lt_trans(j + Nat.1, j + Nat.2, r.length)
        j + Nat.1 < r.length
        lt_imp_lte_suc(j + Nat.1, r.length)
        j + Nat.1 <= r.length
        (j + Nat.1 <= r.length) implies r.drop(j + Nat.1).length + (j + Nat.1) = r.length
        r.drop(j + Nat.1).length + (j + Nat.1) = r.length
        add_comm(r.drop(j + Nat.1).length, j + Nat.1)
        r.drop(j + Nat.1).length + (j + Nat.1) = (j + Nat.1) + r.drop(j + Nat.1).length
        j + Nat.2 < r.length
        j + Nat.2 = j + Nat.1 + Nat.1
        j + Nat.1 + Nat.1 < r.length
        r.length = r.drop(j + Nat.1).length + (j + Nat.1)
        j + Nat.1 + Nat.1 < r.drop(j + Nat.1).length + (j + Nat.1)
        (j + Nat.1) + Nat.1 < (j + Nat.1) + r.drop(j + Nat.1).length
        lt_add_cancel_left(j + Nat.1, Nat.1, r.drop(j + Nat.1).length)
        Nat.1 < r.drop(j + Nat.1).length
        lt_imp_lte_suc(Nat.1, r.drop(j + Nat.1).length)
        Nat.2 <= r.drop(j + Nat.1).length
        Nat.3 <= r.drop(j + Nat.1).length + Nat.1
        Nat.3 <= List.cons(w, r.drop(j + Nat.1)).length
        path_in_vertices_unique(g, s, a, r)
        simple_graph_walk_vertices(a, r).is_unique
        simple_graph_walk_vertices(a, r) = List.cons(a, r)
        List.cons(a, r).is_unique
        unique_implies_tail_unique(a, r)
        r.is_unique
        drop_preserves_unique(r, j + Nat.1)
        r.drop(j + Nat.1).is_unique
        unique_not_contains_drop_suc(r, w)
        r.is_unique and r.contains(w) implies not r.drop(r.find_first_idx(w) + Nat.1).contains(w)
        not r.drop(j + Nat.1).contains(w)
        cons_unique_of_tail_unique_not_contains(w, r.drop(j + Nat.1))
        List.cons(w, r.drop(j + Nat.1)).is_unique
        forall(x: V) {
            if List.cons(w, r.drop(j + Nat.1)).contains(x) {
                List.cons(w, r.drop(j + Nat.1)).contains(x) = (x = w or r.drop(j + Nat.1).contains(x))
                if x = w {
                    s.contains(x)
                }
                if r.drop(j + Nat.1).contains(x) {
                    drop_contains(r, j + Nat.1, x)
                    r.contains(x)
                    path_in_steps_in(g, s, a, r, x)
                    s.contains(x)
                }
                s.contains(x)
            }
            List.cons(w, r.drop(j + Nat.1)).contains(x) implies s.contains(x)
        }
        forall(x: V) { List.cons(w, r.drop(j + Nat.1)).contains(x) implies s.contains(x) }
        simple_graph_cycle(g, walk_last(a, r), List.cons(w, r.drop(j + Nat.1))) =
            (simple_graph_closed_walk(g, walk_last(a, r), List.cons(w, r.drop(j + Nat.1))) and
                Nat.3 <= List.cons(w, r.drop(j + Nat.1)).length and
                List.cons(w, r.drop(j + Nat.1)).is_unique)
        simple_graph_cycle(g, walk_last(a, r), List.cons(w, r.drop(j + Nat.1)))
    }
}

/// The endpoint of a stuck acyclic path of length at least two is a leaf.
theorem path_endpoint_leaf_many[V](g: SimpleGraph[V], s: FiniteSet[V], a: V, r: List[V]) {
    simple_graph_path_in(g, s, a, r) and Nat.2 <= r.length and greedy_stuck(g, s, a, r) and is_acyclic(g, s)
        implies exists(v: V) { s.contains(v) and is_leaf(g, s, v) }
} by {
    if simple_graph_path_in(g, s, a, r) and Nat.2 <= r.length and greedy_stuck(g, s, a, r) and is_acyclic(g, s) {
        path_in_walk(g, s, a, r)
        simple_graph_walk(g, a, walk_last(a, r), r)
        walk_penultimate_adj(g, a, walk_last(a, r), r)
        Nat.1 <= Nat.2
        lte_trans(Nat.1, Nat.2, r.length)
        Nat.1 <= r.length
        g.adj(walk_penultimate(a, r), walk_last(a, r))
        walk_penultimate_vertex(a, r)
        walk_penultimate(a, r) = a or r.contains(walk_penultimate(a, r))
        path_in_start_in(g, s, a, r)
        s.contains(a)
        if walk_penultimate(a, r) = a {
            s.contains(walk_penultimate(a, r))
        }
        if not walk_penultimate(a, r) = a {
            r.contains(walk_penultimate(a, r))
            path_in_steps_in(g, s, a, r, walk_penultimate(a, r))
            s.contains(walk_penultimate(a, r))
        }
        s.contains(walk_penultimate(a, r))
        forall(w: V) {
            if s.contains(w) and g.adj(walk_last(a, r), w) {
                if w = walk_penultimate(a, r) {
                    w = walk_penultimate(a, r)
                }
                if not w = walk_penultimate(a, r) {
                    w != walk_penultimate(a, r)
                    greedy_stuck(g, s, a, r) = forall(u: V) {
                        s.contains(u) and g.adj(walk_last(a, r), u) implies (u = a or r.contains(u))
                    }
                    greedy_stuck(g, s, a, r)
                    w = a or r.contains(w)
                    if w = a {
                        cycle_of_start_edge(g, s, a, r)
                        simple_graph_cycle(g, walk_last(a, r), simple_graph_walk_vertices(a, r))
                        is_acyclic(g, s) = forall(start: V, steps: List[V]) {
                            s.contains(start) and simple_graph_cycle(g, start, steps) and
                                (forall(x: V) { steps.contains(x) implies s.contains(x) })
                                implies false
                        }
                        s.contains(walk_last(a, r))
                        path_in_endpoint_in(g, s, a, r)
                        forall(x: V) {
                            if simple_graph_walk_vertices(a, r).contains(x) {
                                simple_graph_walk_vertices(a, r) = List.cons(a, r)
                                List.cons(a, r).contains(x) = (x = a or r.contains(x))
                                if x = a {
                                    s.contains(x)
                                }
                                if r.contains(x) {
                                    path_in_steps_in(g, s, a, r, x)
                                    s.contains(x)
                                }
                                s.contains(x)
                            }
                            simple_graph_walk_vertices(a, r).contains(x) implies s.contains(x)
                        }
                        s.contains(walk_last(a, r)) and
                            simple_graph_cycle(g, walk_last(a, r), simple_graph_walk_vertices(a, r)) and
                            (forall(x: V) { simple_graph_walk_vertices(a, r).contains(x) implies s.contains(x) })
                        false
                    }
                    not (w = a)
                    r.contains(w)
                    if r.contains(w) {
                        find_first_idx_contains(r, w)
                        r.find_first_idx(w) < r.length
                        if r.find_first_idx(w) + Nat.2 < r.length {
                            cycle_of_inner_edge(g, s, a, r, w, r.find_first_idx(w))
                            simple_graph_cycle(g, walk_last(a, r), List.cons(w, r.drop(r.find_first_idx(w) + Nat.1)))
                            is_acyclic(g, s) = forall(start: V, steps: List[V]) {
                                s.contains(start) and simple_graph_cycle(g, start, steps) and
                                    (forall(x: V) { steps.contains(x) implies s.contains(x) })
                                    implies false
                            }
                            s.contains(walk_last(a, r))
                            path_in_endpoint_in(g, s, a, r)
                            forall(x: V) {
                                if List.cons(w, r.drop(r.find_first_idx(w) + Nat.1)).contains(x) {
                                    List.cons(w, r.drop(r.find_first_idx(w) + Nat.1)).contains(x) =
                                        (x = w or r.drop(r.find_first_idx(w) + Nat.1).contains(x))
                                    if x = w {
                                        s.contains(x)
                                    }
                                    if r.drop(r.find_first_idx(w) + Nat.1).contains(x) {
                                        drop_contains(r, r.find_first_idx(w) + Nat.1, x)
                                        r.contains(x)
                                        path_in_steps_in(g, s, a, r, x)
                                        s.contains(x)
                                    }
                                    s.contains(x)
                                }
                                List.cons(w, r.drop(r.find_first_idx(w) + Nat.1)).contains(x) implies s.contains(x)
                            }
                            s.contains(walk_last(a, r)) and
                                simple_graph_cycle(g, walk_last(a, r), List.cons(w, r.drop(r.find_first_idx(w) + Nat.1))) and
                                (forall(x: V) { List.cons(w, r.drop(r.find_first_idx(w) + Nat.1)).contains(x) implies s.contains(x) })
                            false
                        } else {
                            if r.length = r.find_first_idx(w) + Nat.2 {
                                penultimate_idx(a, r)
                                Nat.2 <= r.length implies r.get_idx(r.length - Nat.2) = Option.some(walk_penultimate(a, r))
                                Nat.2 <= r.length
                                r.get_idx(r.length - Nat.2) = Option.some(walk_penultimate(a, r))
                                add_imp_sub(r.find_first_idx(w), Nat.2, r.length)
                                r.length - Nat.2 = r.find_first_idx(w)
                                r.get_idx(r.find_first_idx(w)) = Option.some(walk_penultimate(a, r))
                                find_first_idx_works(r, w)
                                r.get_idx(r.find_first_idx(w)) = Option.some(w)
                                Option.some(w) = Option.some(walk_penultimate(a, r))
                                w = walk_penultimate(a, r)
                                w != walk_penultimate(a, r)
                                false
                            } else {
                                if r.length = r.find_first_idx(w) + Nat.1 {
                                    drop_length(r, r.find_first_idx(w) + Nat.1)
                                    (r.find_first_idx(w) + Nat.1 <= r.length implies
                                        r.drop(r.find_first_idx(w) + Nat.1).length + (r.find_first_idx(w) + Nat.1) = r.length)
                                    r.length = r.find_first_idx(w) + Nat.1
                                    r.find_first_idx(w) + Nat.1 <= r.length
                                    r.drop(r.find_first_idx(w) + Nat.1).length + (r.find_first_idx(w) + Nat.1) = r.length
                                    r.drop(r.find_first_idx(w) + Nat.1).length + (r.find_first_idx(w) + Nat.1) = r.find_first_idx(w) + Nat.1
                                    Nat.0 + (r.find_first_idx(w) + Nat.1) = r.find_first_idx(w) + Nat.1
                                    r.drop(r.find_first_idx(w) + Nat.1).length + (r.find_first_idx(w) + Nat.1) = Nat.0 + (r.find_first_idx(w) + Nat.1)
                                    add_cancels_right(r.drop(r.find_first_idx(w) + Nat.1).length, Nat.0, r.find_first_idx(w) + Nat.1)
                                    r.drop(r.find_first_idx(w) + Nat.1).length = Nat.0
                                    length_zero_imp_nil(r.drop(r.find_first_idx(w) + Nat.1))
                                    r.drop(r.find_first_idx(w) + Nat.1) = List.nil[V]
                                    walk_suffix_at(g, a, walk_last(a, r), r, w)
                                    simple_graph_walk(g, w, walk_last(a, r), r.drop(r.find_first_idx(w) + Nat.1))
                                    simple_graph_walk(g, w, walk_last(a, r), List.nil[V])
                                    simple_graph_walk_nil(g, w, walk_last(a, r))
                                    w = walk_last(a, r)
                                    g.adj(walk_last(a, r), w)
                                    g.adj(walk_last(a, r), walk_last(a, r))
                                    simple_graph_adj_ne(g, walk_last(a, r), walk_last(a, r))
                                    g.adj(walk_last(a, r), walk_last(a, r)) implies walk_last(a, r) != walk_last(a, r)
                                    false
                                } else {
                                    r.length <= r.find_first_idx(w) + Nat.2
                                    r.find_first_idx(w) + Nat.1 <= r.length
                                    if r.length = r.find_first_idx(w) + Nat.2 {
                                        false
                                    }
                                    not (r.length = r.find_first_idx(w) + Nat.2)
                                    r.find_first_idx(w) + Nat.2 = r.find_first_idx(w) + Nat.1 + Nat.1
                                    r.length <= r.find_first_idx(w) + Nat.1 + Nat.1
                                    r.length != r.find_first_idx(w) + Nat.1 + Nat.1
                                    lte_suc_ne_imp_lte(r.length, r.find_first_idx(w) + Nat.1)
                                    r.length <= r.find_first_idx(w) + Nat.1
                                    lte_antisymm(r.length, r.find_first_idx(w) + Nat.1)
                                    r.length = r.find_first_idx(w) + Nat.1
                                    r.length != r.find_first_idx(w) + Nat.1
                                    false
                                }
                                false
                            }
                            false
                        }
                        false
                    }
                    not r.contains(w)
                    false
                }
                w = walk_penultimate(a, r)
            }
            s.contains(w) and g.adj(walk_last(a, r), w) implies w = walk_penultimate(a, r)
        }
        forall(w: V) {
            s.contains(w) and g.adj(walk_last(a, r), w) implies w = walk_penultimate(a, r)
        }
        s.contains(walk_last(a, r))
        path_in_endpoint_in(g, s, a, r)
        s.contains(walk_penultimate(a, r)) and g.adj(walk_penultimate(a, r), walk_last(a, r)) and
            (forall(w: V) { s.contains(w) and g.adj(walk_last(a, r), w) implies w = walk_penultimate(a, r) })
        is_leaf(g, s, walk_last(a, r)) = exists(u: V) {
            s.contains(u) and g.adj(walk_last(a, r), u) and
                forall(w: V) {
                    s.contains(w) and g.adj(walk_last(a, r), w) implies w = u
                }
        }
        is_leaf(g, s, walk_last(a, r))
        s.contains(walk_last(a, r))
        exists(v: V) { s.contains(v) and is_leaf(g, s, v) }
    }
}

/// The endpoint of a stuck acyclic path is a leaf.
theorem path_endpoint_leaf[V](g: SimpleGraph[V], s: FiniteSet[V], a: V, r: List[V]) {
    simple_graph_path_in(g, s, a, r) and Nat.1 <= r.length and greedy_stuck(g, s, a, r) and is_acyclic(g, s)
        implies exists(v: V) { s.contains(v) and is_leaf(g, s, v) }
} by {
    if simple_graph_path_in(g, s, a, r) and Nat.1 <= r.length and greedy_stuck(g, s, a, r) and is_acyclic(g, s) {
        if r.length = Nat.1 {
            path_endpoint_leaf_one(g, s, a, r)
            exists(v: V) { s.contains(v) and is_leaf(g, s, v) }
        }
        if not r.length = Nat.1 {
            Nat.1 <= r.length
            r.length != Nat.1
            if r.length = Nat.1 {
                r.length != Nat.1
                false
            }
            not (r.length = Nat.1)
            Nat.1 < r.length
            lt_imp_lte_suc(Nat.1, r.length)
            Nat.2 <= r.length
            path_endpoint_leaf_many(g, s, a, r)
            exists(v: V) { s.contains(v) and is_leaf(g, s, v) }
        }
        exists(v: V) { s.contains(v) and is_leaf(g, s, v) }
    }
}

/// Every forest with an edge has a leaf: a vertex of degree one.
theorem forest_has_leaf[V](g: SimpleGraph[V], s: FiniteSet[V]) {
    is_acyclic(g, s) and Nat.1 <= directed_edge_count(g, s) implies
        exists(v: V) { s.contains(v) and is_leaf(g, s, v) }
} by {
    if is_acyclic(g, s) and Nat.1 <= directed_edge_count(g, s) {
        directed_edge_count_eq(g, s)
        directed_edge_count(g, s) = fs_card(directed_edges(g, s))
        fs_member_of_card_pos(directed_edges(g, s))
        exists(p: Pair[V, V]) { directed_edges(g, s).contains(p) }
        let p: Pair[V, V] satisfy { directed_edges(g, s).contains(p) }
        directed_edges_contains_eq(g, s, p)
        directed_edges(g, s).contains(p) = (s.contains(p.first) and s.contains(p.second) and g.adj(p.first, p.second))
        s.contains(p.first)
        s.contains(p.second)
        g.adj(p.first, p.second)
        directed_edges_ne(g, s, p)
        p.first != p.second
        simple_graph_walk_of_adj(g, p.first, p.second)
        simple_graph_walk(g, p.first, p.second, List.singleton(p.second))
        walk_last(p.first, List.singleton(p.second)) = p.second
        simple_graph_walk(g, p.first, walk_last(p.first, List.singleton(p.second)), List.singleton(p.second))
        singleton_unique(p.second)
        List.singleton(p.second).is_unique
        singleton_contains_imp_eq(p.second, p.first)
        List.singleton(p.second).contains(p.first) implies p.first = p.second
        if List.singleton(p.second).contains(p.first) {
            p.first = p.second
            p.first != p.second
            false
        }
        not List.singleton(p.second).contains(p.first)
        cons_unique_of_tail_unique_not_contains(p.first, List.singleton(p.second))
        List.cons(p.first, List.singleton(p.second)).is_unique
        simple_graph_walk_vertices(p.first, List.singleton(p.second)) = List.cons(p.first, List.singleton(p.second))
        simple_graph_walk_vertices(p.first, List.singleton(p.second)).is_unique
        simple_graph_path_intro(g, p.first, p.second, List.singleton(p.second))
        (simple_graph_walk(g, p.first, p.second, List.singleton(p.second)) and
            simple_graph_walk_vertices(p.first, List.singleton(p.second)).is_unique
            implies simple_graph_path(g, p.first, p.second, List.singleton(p.second)))
        simple_graph_path(g, p.first, p.second, List.singleton(p.second))
        walk_last(p.first, List.singleton(p.second)) = p.second
        simple_graph_path(g, p.first, walk_last(p.first, List.singleton(p.second)), List.singleton(p.second))
        forall(x: V) {
            if List.singleton(p.second).contains(x) {
                singleton_contains_imp_eq(p.second, x)
                x = p.second
                s.contains(x)
            }
            List.singleton(p.second).contains(x) implies s.contains(x)
        }
        s.contains(p.first)
        simple_graph_path(g, p.first, walk_last(p.first, List.singleton(p.second)), List.singleton(p.second)) and s.contains(p.first) and
            (forall(x: V) { List.singleton(p.second).contains(x) implies s.contains(x) })
        path_in_intro(g, s, p.first, List.singleton(p.second))
        simple_graph_path_in(g, s, p.first, List.singleton(p.second))
        greedy_path_preserves_path(g, s, p.first, List.singleton(p.second), fs_card(s))
        simple_graph_path_in(g, s, p.first, greedy_path(g, s, p.first, List.singleton(p.second), fs_card(s)))
        singleton_length(p.second)
        List.singleton(p.second).length = Nat.1
        lt_suc(fs_card(s))
        fs_card(s) < fs_card(s).suc
        Nat.1 + fs_card(s) = fs_card(s).suc
        fs_card(s) < Nat.1 + fs_card(s)
        List.singleton(p.second).length + fs_card(s) = Nat.1 + fs_card(s)
        fs_card(s) < List.singleton(p.second).length + fs_card(s)
        greedy_path_maximal(g, s, p.first, List.singleton(p.second), fs_card(s))
        (simple_graph_path_in(g, s, p.first, List.singleton(p.second)) and
            fs_card(s) < List.singleton(p.second).length + fs_card(s)
            implies greedy_stuck(g, s, p.first, greedy_path(g, s, p.first, List.singleton(p.second), fs_card(s))))
        greedy_stuck(g, s, p.first, greedy_path(g, s, p.first, List.singleton(p.second), fs_card(s)))
        greedy_path_length_ge(g, s, p.first, List.singleton(p.second), fs_card(s))
        List.singleton(p.second).length <= greedy_path(g, s, p.first, List.singleton(p.second), fs_card(s)).length
        Nat.1 <= greedy_path(g, s, p.first, List.singleton(p.second), fs_card(s)).length
        path_endpoint_leaf(g, s, p.first, greedy_path(g, s, p.first, List.singleton(p.second), fs_card(s)))
        exists(v: V) { s.contains(v) and is_leaf(g, s, v) }
    }
}

/// Directed edges are monotone in the ambient vertex set.
theorem directed_edges_mono[V](g: SimpleGraph[V], s: FiniteSet[V], t: FiniteSet[V]) {
    s.subset_eq(t) implies directed_edges(g, s).subset_eq(directed_edges(g, t))
} by {
    if s.subset_eq(t) {
        forall(p: Pair[V, V]) {
            if directed_edges(g, s).contains(p) {
                directed_edges_contains_eq(g, s, p)
                directed_edges(g, s).contains(p) =
                    (s.contains(p.first) and s.contains(p.second) and g.adj(p.first, p.second))
                s.contains(p.first)
                s.contains(p.second)
                g.adj(p.first, p.second)
                finite_set_subset_contains(s, t, p.first)
                s.contains(p.first) implies t.contains(p.first)
                t.contains(p.first)
                finite_set_subset_contains(s, t, p.second)
                s.contains(p.second) implies t.contains(p.second)
                t.contains(p.second)
                directed_edges_contains_pair(g, t, p.first, p.second)
                (t.contains(p.first) and t.contains(p.second) and g.adj(p.first, p.second) implies
                    directed_edges(g, t).contains(Pair.new(p.first, p.second)))
                directed_edges(g, t).contains(Pair.new(p.first, p.second))
                Pair.new(p.first, p.second) = p
                directed_edges(g, t).contains(p)
            }
            directed_edges(g, s).contains(p) implies directed_edges(g, t).contains(p)
        }
        fs_subset_eq_intro(directed_edges(g, s), directed_edges(g, t))
        directed_edges(g, s).subset_eq(directed_edges(g, t))
    }
}

/// Removing a leaf removes exactly its two directed edges.
theorem edges_remove_leaf[V](g: SimpleGraph[V], s: FiniteSet[V], v: V) {
    is_leaf(g, s, v) and s.contains(v) implies
        directed_edge_count(g, s) = directed_edge_count(g, fs_remove(s, v)) + Nat.2
} by {
    if is_leaf(g, s, v) and s.contains(v) {
        is_leaf(g, s, v) = exists(u: V) {
            s.contains(u) and g.adj(v, u) and
                forall(w: V) {
                    s.contains(w) and g.adj(v, w) implies w = u
                }
        }
        exists(u: V) {
            s.contains(u) and g.adj(v, u) and
                forall(w: V) {
                    s.contains(w) and g.adj(v, w) implies w = u
                }
        }
        let u: V satisfy {
            s.contains(u) and g.adj(v, u) and
                forall(w: V) {
                    s.contains(w) and g.adj(v, w) implies w = u
                }
        }
        s.contains(u)
        g.adj(v, u)
        simple_graph_adj_ne(g, v, u)
        g.adj(v, u) implies v != u
        v != u
        s.contains(v)
        // s' = fs_remove(s, v)
        fs_remove_contains_eq(s, v, v)
        fs_remove(s, v).contains(v) = (s.contains(v) and v != v)
        not fs_remove(s, v).contains(v)
        // edges(s) = edges(s') ∪ {(v,u)} ∪ {(u,v)}:
        forall(p: Pair[V, V]) {
            if directed_edges(g, s).contains(p) {
                directed_edges_contains_eq(g, s, p)
                directed_edges(g, s).contains(p) =
                    (s.contains(p.first) and s.contains(p.second) and g.adj(p.first, p.second))
                s.contains(p.first)
                s.contains(p.second)
                g.adj(p.first, p.second)
                if p.first = v {
                    forall(w: V) {
                        s.contains(w) and g.adj(v, w) implies w = u
                    }
                    (s.contains(p.second) and g.adj(v, p.second) implies p.second = u)
                    p.second = u
                    Pair.new(p.first, p.second) = Pair.new(v, u)
                    fs_insert_contains_self(directed_edges(g, fs_remove(s, v)), Pair.new(v, u))
                    fs_insert(directed_edges(g, fs_remove(s, v)), Pair.new(v, u)).contains(Pair.new(v, u))
                    fs_insert_contains_eq(directed_edges(g, fs_remove(s, v)), Pair.new(v, u), p)
                    fs_insert(directed_edges(g, fs_remove(s, v)), Pair.new(v, u)).contains(p) =
                        (p = Pair.new(v, u) or directed_edges(g, fs_remove(s, v)).contains(p))
                    p = Pair.new(v, u)
                    fs_insert(directed_edges(g, fs_remove(s, v)), Pair.new(v, u)).contains(p)
                    fs_insert_contains_of_contains(fs_insert(directed_edges(g, fs_remove(s, v)), Pair.new(v, u)), Pair.new(u, v), p)
                    fs_insert(fs_insert(directed_edges(g, fs_remove(s, v)), Pair.new(v, u)), Pair.new(u, v)).contains(p)
                }
                if not p.first = v {
                    p.first != v
                    if p.second = v {
                        simple_graph_adj_comm(g, p.first, p.second)
                        g.adj(p.second, p.first)
                        g.adj(v, p.first)
                        forall(w: V) {
                            s.contains(w) and g.adj(v, w) implies w = u
                        }
                        (s.contains(p.first) and g.adj(v, p.first) implies p.first = u)
                        p.first = u
                        Pair.new(p.first, p.second) = Pair.new(u, v)
                        p = Pair.new(u, v)
                        fs_insert_contains_self(fs_insert(directed_edges(g, fs_remove(s, v)), Pair.new(v, u)), Pair.new(u, v))
                        fs_insert(fs_insert(directed_edges(g, fs_remove(s, v)), Pair.new(v, u)), Pair.new(u, v)).contains(Pair.new(u, v))
                        (p = Pair.new(u, v) implies
                            fs_insert(fs_insert(directed_edges(g, fs_remove(s, v)), Pair.new(v, u)), Pair.new(u, v)).contains(p))
                        fs_insert(fs_insert(directed_edges(g, fs_remove(s, v)), Pair.new(v, u)), Pair.new(u, v)).contains(p)
                    }
                    if not p.second = v {
                        p.second != v
                        fs_remove_contains_eq(s, v, p.first)
                        fs_remove(s, v).contains(p.first) = (s.contains(p.first) and p.first != v)
                        fs_remove(s, v).contains(p.first)
                        fs_remove_contains_eq(s, v, p.second)
                        fs_remove(s, v).contains(p.second) = (s.contains(p.second) and p.second != v)
                        fs_remove(s, v).contains(p.second)
                        directed_edges_contains_pair(g, fs_remove(s, v), p.first, p.second)
                        directed_edges(g, fs_remove(s, v)).contains(Pair.new(p.first, p.second))
                        fs_insert_contains_of_contains(directed_edges(g, fs_remove(s, v)), Pair.new(v, u), p)
                        fs_insert(directed_edges(g, fs_remove(s, v)), Pair.new(v, u)).contains(p)
                        fs_insert_contains_of_contains(fs_insert(directed_edges(g, fs_remove(s, v)), Pair.new(v, u)), Pair.new(u, v), p)
                        fs_insert(fs_insert(directed_edges(g, fs_remove(s, v)), Pair.new(v, u)), Pair.new(u, v)).contains(p)
                    }
                    fs_insert(fs_insert(directed_edges(g, fs_remove(s, v)), Pair.new(v, u)), Pair.new(u, v)).contains(p)
                }
                fs_insert(fs_insert(directed_edges(g, fs_remove(s, v)), Pair.new(v, u)), Pair.new(u, v)).contains(p)
            }
            (directed_edges(g, s).contains(p) implies
                fs_insert(fs_insert(directed_edges(g, fs_remove(s, v)), Pair.new(v, u)), Pair.new(u, v)).contains(p))
        }
        forall(p: Pair[V, V]) {
            if fs_insert(fs_insert(directed_edges(g, fs_remove(s, v)), Pair.new(v, u)), Pair.new(u, v)).contains(p) {
                fs_insert_contains_eq(fs_insert(directed_edges(g, fs_remove(s, v)), Pair.new(v, u)), Pair.new(u, v), p)
                fs_insert(fs_insert(directed_edges(g, fs_remove(s, v)), Pair.new(v, u)), Pair.new(u, v)).contains(p) =
                    (p = Pair.new(u, v) or fs_insert(directed_edges(g, fs_remove(s, v)), Pair.new(v, u)).contains(p))
                if p = Pair.new(u, v) {
                    fs_remove_subset(s, v)
                    fs_remove(s, v).subset_eq(s)
                    directed_edges_contains_pair(g, s, u, v)
                    (s.contains(u) and s.contains(v) and g.adj(u, v) implies
                        directed_edges(g, s).contains(Pair.new(u, v)))
                    simple_graph_adj_comm(g, v, u)
                    g.adj(u, v)
                    directed_edges(g, s).contains(Pair.new(u, v))
                    Pair.new(u, v) = p
                    directed_edges(g, s).contains(p)
                }
                if not p = Pair.new(u, v) {
                    fs_insert(directed_edges(g, fs_remove(s, v)), Pair.new(v, u)).contains(p)
                    fs_insert_contains_eq(directed_edges(g, fs_remove(s, v)), Pair.new(v, u), p)
                    fs_insert(directed_edges(g, fs_remove(s, v)), Pair.new(v, u)).contains(p) =
                        (p = Pair.new(v, u) or directed_edges(g, fs_remove(s, v)).contains(p))
                    if p = Pair.new(v, u) {
                        directed_edges_contains_pair(g, s, v, u)
                        (s.contains(v) and s.contains(u) and g.adj(v, u) implies
                            directed_edges(g, s).contains(Pair.new(v, u)))
                        directed_edges(g, s).contains(Pair.new(v, u))
                        Pair.new(v, u) = p
                        directed_edges(g, s).contains(p)
                    }
                    if not p = Pair.new(v, u) {
                        directed_edges(g, fs_remove(s, v)).contains(p)
                        directed_edges_mono(g, fs_remove(s, v), s)
                        fs_remove_subset(s, v)
                        fs_remove(s, v).subset_eq(s)
                        (fs_remove(s, v).subset_eq(s) implies
                            directed_edges(g, fs_remove(s, v)).subset_eq(directed_edges(g, s)))
                        directed_edges(g, fs_remove(s, v)).subset_eq(directed_edges(g, s))
                        finite_set_subset_contains(directed_edges(g, fs_remove(s, v)), directed_edges(g, s), p)
                        (directed_edges(g, fs_remove(s, v)).subset_eq(directed_edges(g, s)) implies
                            (directed_edges(g, fs_remove(s, v)).contains(p) implies directed_edges(g, s).contains(p)))
                        directed_edges(g, fs_remove(s, v)).contains(p) implies directed_edges(g, s).contains(p)
                        directed_edges(g, s).contains(p)
                    }
                    directed_edges(g, s).contains(p)
                }
                directed_edges(g, s).contains(p)
            }
            (fs_insert(fs_insert(directed_edges(g, fs_remove(s, v)), Pair.new(v, u)), Pair.new(u, v)).contains(p) implies
                directed_edges(g, s).contains(p))
        }
        fs_subset_eq_intro(directed_edges(g, s),
            fs_insert(fs_insert(directed_edges(g, fs_remove(s, v)), Pair.new(v, u)), Pair.new(u, v)))
        directed_edges(g, s).subset_eq(
            fs_insert(fs_insert(directed_edges(g, fs_remove(s, v)), Pair.new(v, u)), Pair.new(u, v)))
        fs_subset_eq_intro(
            fs_insert(fs_insert(directed_edges(g, fs_remove(s, v)), Pair.new(v, u)), Pair.new(u, v)),
            directed_edges(g, s))
        fs_insert(fs_insert(directed_edges(g, fs_remove(s, v)), Pair.new(v, u)), Pair.new(u, v)).subset_eq(directed_edges(g, s))
        fs_card_eq_of_mutual_subset(directed_edges(g, s),
            fs_insert(fs_insert(directed_edges(g, fs_remove(s, v)), Pair.new(v, u)), Pair.new(u, v)))
        fs_card(directed_edges(g, s)) = fs_card(
            fs_insert(fs_insert(directed_edges(g, fs_remove(s, v)), Pair.new(v, u)), Pair.new(u, v)))
        // |{(v,u)} ∪ {(u,v)}| = 2 and disjoint from edges(s'):
        fs_card_insert_of_not_contains(FiniteSet.empty[Pair[V, V]], Pair.new(v, u))
        fs_card(fs_insert(FiniteSet.empty[Pair[V, V]], Pair.new(v, u))) =
            fs_card(FiniteSet.empty[Pair[V, V]]) + Nat.1
        fs_card_empty[Pair[V, V]]
        fs_card(FiniteSet.empty[Pair[V, V]]) = Nat.0
        fs_card(fs_insert(FiniteSet.empty[Pair[V, V]], Pair.new(v, u))) = Nat.1
        fs_insert_contains_eq(FiniteSet.empty[Pair[V, V]], Pair.new(v, u), Pair.new(u, v))
        fs_insert(FiniteSet.empty[Pair[V, V]], Pair.new(v, u)).contains(Pair.new(u, v)) =
            (Pair.new(u, v) = Pair.new(v, u) or FiniteSet.empty[Pair[V, V]].contains(Pair.new(u, v)))
        if Pair.new(u, v) = Pair.new(v, u) {
            Pair.new(u, v).first = Pair.new(v, u).first
            u = v
            v != u
            false
        }
        not fs_insert(FiniteSet.empty[Pair[V, V]], Pair.new(v, u)).contains(Pair.new(u, v))
        fs_card_insert_of_not_contains(fs_insert(FiniteSet.empty[Pair[V, V]], Pair.new(v, u)), Pair.new(u, v))
        fs_card(fs_insert(fs_insert(FiniteSet.empty[Pair[V, V]], Pair.new(v, u)), Pair.new(u, v))) =
            fs_card(fs_insert(FiniteSet.empty[Pair[V, V]], Pair.new(v, u))) + Nat.1
        fs_card(fs_insert(fs_insert(FiniteSet.empty[Pair[V, V]], Pair.new(v, u)), Pair.new(u, v))) = Nat.2
        // (v, u) not in edges(s'):
        directed_edges_contains_eq(g, fs_remove(s, v), Pair.new(v, u))
        directed_edges(g, fs_remove(s, v)).contains(Pair.new(v, u)) =
            (fs_remove(s, v).contains(v) and fs_remove(s, v).contains(u) and g.adj(v, u))
        if directed_edges(g, fs_remove(s, v)).contains(Pair.new(v, u)) {
            fs_remove(s, v).contains(v)
            not fs_remove(s, v).contains(v)
            false
        }
        not directed_edges(g, fs_remove(s, v)).contains(Pair.new(v, u))
        // (u, v) not in edges(s'):
        directed_edges_contains_eq(g, fs_remove(s, v), Pair.new(u, v))
        directed_edges(g, fs_remove(s, v)).contains(Pair.new(u, v)) =
            (fs_remove(s, v).contains(u) and fs_remove(s, v).contains(v) and g.adj(u, v))
        if directed_edges(g, fs_remove(s, v)).contains(Pair.new(u, v)) {
            fs_remove(s, v).contains(v)
            not fs_remove(s, v).contains(v)
            false
        }
        not directed_edges(g, fs_remove(s, v)).contains(Pair.new(u, v))
        // (u, v) not in {(v, u)}:
        fs_insert_contains_eq(FiniteSet.empty[Pair[V, V]], Pair.new(v, u), Pair.new(u, v))
        fs_insert(FiniteSet.empty[Pair[V, V]], Pair.new(v, u)).contains(Pair.new(u, v)) =
            (Pair.new(u, v) = Pair.new(v, u) or FiniteSet.empty[Pair[V, V]].contains(Pair.new(u, v)))
        if Pair.new(u, v) = Pair.new(v, u) {
            Pair.new(u, v).first = Pair.new(v, u).first
            u = v
            v != u
            false
        }
        not fs_insert(FiniteSet.empty[Pair[V, V]], Pair.new(v, u)).contains(Pair.new(u, v))
        // |{(v,u)}| = 1 and |{(v,u),(u,v)}| = 2:
        fs_card_insert_of_not_contains(FiniteSet.empty[Pair[V, V]], Pair.new(v, u))
        fs_card(fs_insert(FiniteSet.empty[Pair[V, V]], Pair.new(v, u))) =
            fs_card(FiniteSet.empty[Pair[V, V]]) + Nat.1
        fs_card_empty[Pair[V, V]]
        fs_card(FiniteSet.empty[Pair[V, V]]) = Nat.0
        fs_card(fs_insert(FiniteSet.empty[Pair[V, V]], Pair.new(v, u))) = Nat.1
        fs_card_insert_of_not_contains(fs_insert(FiniteSet.empty[Pair[V, V]], Pair.new(v, u)), Pair.new(u, v))
        fs_card(fs_insert(fs_insert(FiniteSet.empty[Pair[V, V]], Pair.new(v, u)), Pair.new(u, v))) =
            fs_card(fs_insert(FiniteSet.empty[Pair[V, V]], Pair.new(v, u))) + Nat.1
        fs_card(fs_insert(fs_insert(FiniteSet.empty[Pair[V, V]], Pair.new(v, u)), Pair.new(u, v))) = Nat.2
        // |edges(s)| = |fs_insert(fs_insert(edges(s'), (v,u)), (u,v))|:
        fs_card_eq_of_mutual_subset(directed_edges(g, s),
            fs_insert(fs_insert(directed_edges(g, fs_remove(s, v)), Pair.new(v, u)), Pair.new(u, v)))
        fs_card(directed_edges(g, s)) = fs_card(
            fs_insert(fs_insert(directed_edges(g, fs_remove(s, v)), Pair.new(v, u)), Pair.new(u, v)))
        // |fs_insert(fs_insert(E, a), b)| = |E| + 1 + 1:
        fs_card_insert_of_not_contains(directed_edges(g, fs_remove(s, v)), Pair.new(v, u))
        fs_card(fs_insert(directed_edges(g, fs_remove(s, v)), Pair.new(v, u))) =
            fs_card(directed_edges(g, fs_remove(s, v))) + Nat.1
        fs_card_insert_of_not_contains(fs_insert(directed_edges(g, fs_remove(s, v)), Pair.new(v, u)), Pair.new(u, v))
        fs_card(fs_insert(fs_insert(directed_edges(g, fs_remove(s, v)), Pair.new(v, u)), Pair.new(u, v))) =
            fs_card(fs_insert(directed_edges(g, fs_remove(s, v)), Pair.new(v, u))) + Nat.1
        fs_card(fs_insert(fs_insert(directed_edges(g, fs_remove(s, v)), Pair.new(v, u)), Pair.new(u, v))) =
            fs_card(directed_edges(g, fs_remove(s, v))) + Nat.2
        fs_card(directed_edges(g, s)) =
            fs_card(directed_edges(g, fs_remove(s, v))) + Nat.2
        directed_edge_count_eq(g, s)
        directed_edge_count(g, s) = fs_card(directed_edges(g, s))
        directed_edge_count_eq(g, fs_remove(s, v))
        directed_edge_count(g, fs_remove(s, v)) = fs_card(directed_edges(g, fs_remove(s, v)))
        directed_edge_count(g, s) = directed_edge_count(g, fs_remove(s, v)) + Nat.2
    }
}

/// A set with no members has cardinality zero.
theorem fs_card_zero_of_no_members[T](s: FiniteSet[T]) {
    (forall(x: T) { not s.contains(x) }) implies fs_card(s) = Nat.0
} by {
    if forall(x: T) { not s.contains(x) } {
        finite_set_empty_or_inhabited(s)
        s = FiniteSet.empty[T] or exists(x: T) { s.contains(x) }
        if s = FiniteSet.empty[T] {
            fs_card_empty[T]
            fs_card(FiniteSet.empty[T]) = Nat.0
            fs_card(s) = Nat.0
        }
        if not s = FiniteSet.empty[T] {
            finite_set_has_member_of_ne_empty(s)
            exists(x: T) { s.contains(x) }
            let x: T satisfy { s.contains(x) }
            not s.contains(x)
            false
        }
        fs_card(s) = Nat.0
    }
}

/// Not being at least one means being zero.
theorem not_one_lte_imp_zero(x: Nat) {
    not (Nat.1 <= x) implies x = Nat.0
} by {
    if not (Nat.1 <= x) {
        if x = Nat.0 {
            x = Nat.0
        }
        if not x = Nat.0 {
            x != Nat.0
            pos_of_ne_zero(x)
            Nat.0 < x
            lt_imp_lte_suc(Nat.0, x)
            Nat.1 <= x
            not (Nat.1 <= x)
            false
        }
        x = Nat.0
    }
}

/// The edge-bound claim at a fixed vertex count.
define forest_edges_bound_at[V](g: SimpleGraph[V], n: Nat) -> Bool {
    forall(s: FiniteSet[V]) {
        fs_card(s) = n implies (is_acyclic(g, s) implies
            directed_edge_count(g, s) <= Nat.2 * fs_card(s) - Nat.2)
    }
}

/// A forest on `n` vertices has at most `n - 1` edges.
theorem forest_edges_bound[V](g: SimpleGraph[V], s: FiniteSet[V]) {
    is_acyclic(g, s) implies directed_edge_count(g, s) <= Nat.2 * fs_card(s) - Nat.2
} by {
    forall(s2: FiniteSet[V]) {
        if fs_card(s2) = Nat.0 {
            forall(p: Pair[V, V]) {
                if directed_edges(g, s2).contains(p) {
                    directed_edges_contains_eq(g, s2, p)
                    directed_edges(g, s2).contains(p) =
                        (s2.contains(p.first) and s2.contains(p.second) and g.adj(p.first, p.second))
                    s2.contains(p.first)
                    fs_card_pos_of_contains(s2, p.first)
                    Nat.1 <= fs_card(s2)
                    fs_card(s2) = Nat.0
                    Nat.1 <= Nat.0
                    only_zero_lte_zero(Nat.1)
                    Nat.1 = Nat.0
                    false
                }
                not directed_edges(g, s2).contains(p)
            }
            fs_card_zero_of_no_members(directed_edges(g, s2))
            fs_card(directed_edges(g, s2)) = Nat.0
            directed_edge_count_eq(g, s2)
            directed_edge_count(g, s2) = fs_card(directed_edges(g, s2))
            directed_edge_count(g, s2) = Nat.0
            Nat.0 <= Nat.2 * Nat.0 - Nat.2
            Nat.2 * fs_card(s2) - Nat.2 = Nat.2 * Nat.0 - Nat.2
            directed_edge_count(g, s2) <= Nat.2 * fs_card(s2) - Nat.2
        }
        fs_card(s2) = Nat.0 implies (is_acyclic(g, s2) implies
            directed_edge_count(g, s2) <= Nat.2 * fs_card(s2) - Nat.2)
    }
    forest_edges_bound_at(g, Nat.0) = forall(s3: FiniteSet[V]) {
        fs_card(s3) = Nat.0 implies (is_acyclic(g, s3) implies
            directed_edge_count(g, s3) <= Nat.2 * fs_card(s3) - Nat.2)
    }
    forest_edges_bound_at(g, Nat.0)
    forall(n: Nat) {
        if forest_edges_bound_at(g, n) {
            forall(s2: FiniteSet[V]) {
                if fs_card(s2) = n + Nat.1 {
                    if is_acyclic(g, s2) {
                        if Nat.1 <= directed_edge_count(g, s2) {
                            directed_edge_count_eq(g, s2)
                            directed_edge_count(g, s2) = fs_card(directed_edges(g, s2))
                            fs_member_of_card_pos(directed_edges(g, s2))
                            exists(p: Pair[V, V]) { directed_edges(g, s2).contains(p) }
                            let p: Pair[V, V] satisfy { directed_edges(g, s2).contains(p) }
                            directed_edges_contains_eq(g, s2, p)
                            directed_edges(g, s2).contains(p) =
                                (s2.contains(p.first) and s2.contains(p.second) and g.adj(p.first, p.second))
                            s2.contains(p.first)
                            s2.contains(p.second)
                            g.adj(p.first, p.second)
                            directed_edges_ne(g, s2, p)
                            p.first != p.second
                            fs_card_two_of_distinct_members(s2, p.first, p.second)
                            Nat.2 <= fs_card(s2)
                            fs_card(s2) = n + Nat.1
                            Nat.2 <= n + Nat.1
                            n + Nat.1 = n.suc
                            Nat.2 = Nat.1.suc
                            Nat.1.suc <= n.suc
                            lte_cancel_suc(Nat.1, n)
                            Nat.1 <= n
                            forest_has_leaf(g, s2)
                            ((is_acyclic(g, s2) and Nat.1 <= directed_edge_count(g, s2)) implies
                                exists(v: V) { s2.contains(v) and is_leaf(g, s2, v) })
                            exists(v: V) { s2.contains(v) and is_leaf(g, s2, v) }
                            let v: V satisfy { s2.contains(v) and is_leaf(g, s2, v) }
                            s2.contains(v)
                            is_leaf(g, s2, v)
                            edges_remove_leaf(g, s2, v)
                            ((is_leaf(g, s2, v) and s2.contains(v)) implies
                                directed_edge_count(g, s2) = directed_edge_count(g, fs_remove(s2, v)) + Nat.2)
                            directed_edge_count(g, s2) = directed_edge_count(g, fs_remove(s2, v)) + Nat.2
                            fs_card_remove_of_contains(s2, v)
                            fs_card(s2) = fs_card(fs_remove(s2, v)) + Nat.1
                            fs_card(s2) = n + Nat.1
                            fs_card(fs_remove(s2, v)) + Nat.1 = n + Nat.1
                            add_cancels_left(fs_card(fs_remove(s2, v)), n, Nat.1)
                            fs_card(fs_remove(s2, v)) = n
                            is_acyclic_subset(g, s2, fs_remove(s2, v))
                            fs_remove_subset(s2, v)
                            fs_remove(s2, v).subset_eq(s2)
                            is_acyclic(g, fs_remove(s2, v))
                            forest_edges_bound_at(g, n) = forall(s3: FiniteSet[V]) {
                                fs_card(s3) = n implies (is_acyclic(g, s3) implies
                                    directed_edge_count(g, s3) <= Nat.2 * fs_card(s3) - Nat.2)
                            }
                            forest_edges_bound_at(g, n)
                            fs_card(fs_remove(s2, v)) = n
                            is_acyclic(g, fs_remove(s2, v))
                            directed_edge_count(g, fs_remove(s2, v)) <= Nat.2 * fs_card(fs_remove(s2, v)) - Nat.2
                            fs_card(fs_remove(s2, v)) = n
                            Nat.2 * fs_card(fs_remove(s2, v)) - Nat.2 = Nat.2 * n - Nat.2
                            directed_edge_count(g, fs_remove(s2, v)) <= Nat.2 * n - Nat.2
                            directed_edge_count(g, fs_remove(s2, v)) + Nat.2 <= Nat.2 * n - Nat.2 + Nat.2
                            directed_edge_count(g, s2) <= Nat.2 * n - Nat.2 + Nat.2
                            Nat.1 <= n
                            Nat.2 <= Nat.2 * n
                            add_sub(Nat.2 * n, Nat.2)
                            Nat.2 <= Nat.2 * n implies Nat.2 * n - Nat.2 + Nat.2 = Nat.2 * n
                            Nat.2 * n - Nat.2 + Nat.2 = Nat.2 * n
                            Nat.2 * (n + Nat.1) = Nat.2 * n + Nat.2
                            Nat.2 * n + Nat.2 - Nat.2 = Nat.2 * n
                            add_imp_sub(Nat.2 * n, Nat.2, Nat.2 * (n + Nat.1))
                            Nat.2 * (n + Nat.1) - Nat.2 = Nat.2 * n
                            directed_edge_count(g, s2) <= Nat.2 * (n + Nat.1) - Nat.2
                            fs_card(s2) = n + Nat.1
                            Nat.2 * fs_card(s2) - Nat.2 = Nat.2 * (n + Nat.1) - Nat.2
                            directed_edge_count(g, s2) <= Nat.2 * fs_card(s2) - Nat.2
                        }
                        if not Nat.1 <= directed_edge_count(g, s2) {
                            not_one_lte_imp_zero(directed_edge_count(g, s2))
                            directed_edge_count(g, s2) = Nat.0
                            Nat.0 <= Nat.2 * fs_card(s2) - Nat.2
                            directed_edge_count(g, s2) <= Nat.2 * fs_card(s2) - Nat.2
                        }
                        directed_edge_count(g, s2) <= Nat.2 * fs_card(s2) - Nat.2
                    }
                    is_acyclic(g, s2) implies directed_edge_count(g, s2) <= Nat.2 * fs_card(s2) - Nat.2
                }
                fs_card(s2) = n + Nat.1 implies (is_acyclic(g, s2) implies
                    directed_edge_count(g, s2) <= Nat.2 * fs_card(s2) - Nat.2)
            }
            forest_edges_bound_at(g, n + Nat.1) = forall(s3: FiniteSet[V]) {
                fs_card(s3) = n + Nat.1 implies (is_acyclic(g, s3) implies
                    directed_edge_count(g, s3) <= Nat.2 * fs_card(s3) - Nat.2)
            }
            forest_edges_bound_at(g, n + Nat.1)
        }
    }
    forall(n: Nat) { forest_edges_bound_at(g, n) implies forest_edges_bound_at(g, n + Nat.1) }
    forall(n: Nat) { forest_edges_bound_at(g, n) implies forest_edges_bound_at(g, n.suc) }
    forest_edges_bound_at(g, Nat.0) and
        forall(n: Nat) { forest_edges_bound_at(g, n) implies forest_edges_bound_at(g, n.suc) }
    alt_induction(forest_edges_bound_at(g))
    forall(n: Nat) { forest_edges_bound_at(g, n) }
    forest_edges_bound_at(g, fs_card(s))
    forest_edges_bound_at(g, fs_card(s)) = forall(s3: FiniteSet[V]) {
        fs_card(s3) = fs_card(s) implies (is_acyclic(g, s3) implies
            directed_edge_count(g, s3) <= Nat.2 * fs_card(s3) - Nat.2)
    }
    fs_card(s) = fs_card(s)
    is_acyclic(g, s) implies directed_edge_count(g, s) <= Nat.2 * fs_card(s) - Nat.2
}

/// Reachability inside a smaller vertex set lifts to a larger one.
theorem reachable_mono_on[V](g: SimpleGraph[V], s: FiniteSet[V], t: FiniteSet[V], x: V, y: V) {
    t.subset_eq(s) and simple_graph_reachable(induced_subgraph(g, t.underlying_set), x, y)
        implies simple_graph_reachable(induced_subgraph(g, s.underlying_set), x, y)
} by {
    if t.subset_eq(s) and simple_graph_reachable(induced_subgraph(g, t.underlying_set), x, y) {
        forall(a: V, b: V) {
            if induced_subgraph(g, t.underlying_set).adj(a, b) {
                induced_subgraph_adj_eq(g, t.underlying_set, a, b)
                induced_subgraph(g, t.underlying_set).adj(a, b) =
                    (g.adj(a, b) and t.contains(a) and t.contains(b))
                g.adj(a, b)
                t.contains(a)
                t.contains(b)
                finite_set_subset_contains(t, s, a)
                t.contains(a) implies s.contains(a)
                s.contains(a)
                finite_set_subset_contains(t, s, b)
                t.contains(b) implies s.contains(b)
                s.contains(b)
                induced_subgraph_adj_eq(g, s.underlying_set, a, b)
                induced_subgraph(g, s.underlying_set).adj(a, b) =
                    (g.adj(a, b) and s.contains(a) and s.contains(b))
                induced_subgraph(g, s.underlying_set).adj(a, b)
            }
            (induced_subgraph(g, t.underlying_set).adj(a, b) implies
                induced_subgraph(g, s.underlying_set).adj(a, b))
        }
        relation_subset(induced_subgraph(g, t.underlying_set).adj, induced_subgraph(g, s.underlying_set).adj)
        relation_refl_trans_closure_monotone(induced_subgraph(g, t.underlying_set).adj,
            induced_subgraph(g, s.underlying_set).adj)
        relation_subset(
            relation_refl_trans_closure(induced_subgraph(g, t.underlying_set).adj),
            relation_refl_trans_closure(induced_subgraph(g, s.underlying_set).adj))
        simple_graph_reachable(induced_subgraph(g, t.underlying_set), x, y) =
            relation_refl_trans_closure(induced_subgraph(g, t.underlying_set).adj, x, y)
        relation_refl_trans_closure(induced_subgraph(g, t.underlying_set).adj, x, y)
        relation_subset_step(
            relation_refl_trans_closure(induced_subgraph(g, t.underlying_set).adj),
            relation_refl_trans_closure(induced_subgraph(g, s.underlying_set).adj), x, y)
        relation_refl_trans_closure(induced_subgraph(g, s.underlying_set).adj, x, y)
        simple_graph_reachable(induced_subgraph(g, s.underlying_set), x, y) =
            relation_refl_trans_closure(induced_subgraph(g, s.underlying_set).adj, x, y)
        simple_graph_reachable(induced_subgraph(g, s.underlying_set), x, y)
    }
}

/// A set of cardinality one has equal members.
theorem fs_card_one_members_eq[T](s: FiniteSet[T]) {
    fs_card(s) = Nat.1 implies (forall(x: T, y: T) { s.contains(x) and s.contains(y) implies x = y })
} by {
    if fs_card(s) = Nat.1 {
        finite_set_cardinality_has_exact_unique_list(s, Nat.1)
        exists(rep: List[T]) { fs_from_list(rep) = s and rep.is_unique and rep.length = fs_card(s) }
        let rep: List[T] satisfy { fs_from_list(rep) = s and rep.is_unique and rep.length = fs_card(s) }
        rep.length = Nat.1
        match rep {
            List.nil[T] {
                List.nil[T].length = Nat.0
                Nat.1 = Nat.0
                false
            }
            List.cons(head, tail) {
                List.cons(head, tail).length = tail.length + Nat.1
                rep.length = tail.length + Nat.1
                tail.length + Nat.1 = Nat.1
                tail.length = Nat.0
                tail = List.nil[T]
                rep = List.cons(head, List.nil[T])
                forall(x: T, y: T) {
                    if s.contains(x) and s.contains(y) {
                        fs_from_list_contains_eq(rep, x)
                        fs_from_list(rep).contains(x) = rep.contains(x)
                        fs_from_list(rep) = s
                        s.contains(x) = rep.contains(x)
                        rep.contains(x)
                        List.cons(head, List.nil[T]).contains(x) = (x = head or List.nil[T].contains(x))
                        x = head
                        fs_from_list_contains_eq(rep, y)
                        fs_from_list(rep).contains(y) = rep.contains(y)
                        s.contains(y) = rep.contains(y)
                        rep.contains(y)
                        List.cons(head, List.nil[T]).contains(y) = (y = head or List.nil[T].contains(y))
                        y = head
                        x = y
                    }
                    s.contains(x) and s.contains(y) implies x = y
                }
                forall(x: T, y: T) { s.contains(x) and s.contains(y) implies x = y }
            }
        }
        forall(x: T, y: T) { s.contains(x) and s.contains(y) implies x = y }
    }
}

/// The tree-of-edge-count claim at a fixed vertex count.
define forest_tree_at[V](g: SimpleGraph[V], n: Nat) -> Bool {
    forall(s: FiniteSet[V]) {
        fs_card(s) = n implies
            ((is_acyclic(g, s) and Nat.1 <= fs_card(s) and
                directed_edge_count(g, s) = Nat.2 * fs_card(s) - Nat.2)
            implies simple_graph_connected_on(g, s))
    }
}

/// Two times a number of size at least two minus two is at least one.
theorem two_mul_sub_two_pos(n: Nat) {
    Nat.2 <= n implies Nat.1 <= Nat.2 * n - Nat.2
} by {
    if Nat.2 <= n {
        lte_cancel_suc(Nat.1, n - Nat.1)
        Nat.1.suc <= (n - Nat.1).suc implies Nat.1 <= n - Nat.1
        Nat.1 <= Nat.2
        lte_trans(Nat.1, Nat.2, n)
        Nat.1 <= n
        add_sub(n, Nat.1)
        Nat.1 <= n implies n - Nat.1 + Nat.1 = n
        n - Nat.1 + Nat.1 = n
        (n - Nat.1).suc = n
        Nat.2 = Nat.1.suc
        Nat.1.suc <= n
        Nat.1.suc <= (n - Nat.1).suc
        Nat.1 <= n - Nat.1
        lte_mul_both(Nat.2, Nat.1, n - Nat.1)
        Nat.1 <= n - Nat.1 implies Nat.2 * Nat.1 <= Nat.2 * (n - Nat.1)
        Nat.2 * Nat.1 <= Nat.2 * (n - Nat.1)
        Nat.2 * Nat.1 = Nat.2
        Nat.2 <= Nat.2 * (n - Nat.1)
        Nat.2 * (n - Nat.1 + Nat.1) = Nat.2 * n
        Nat.2 * (n - Nat.1) + Nat.2 = Nat.2 * n
        add_imp_sub(Nat.2 * (n - Nat.1), Nat.2, Nat.2 * n)
        Nat.2 * n - Nat.2 = Nat.2 * (n - Nat.1)
        Nat.2 * (n - Nat.1) = Nat.2 * n - Nat.2
        Nat.2 <= Nat.2 * n - Nat.2
        lte_trans(Nat.1, Nat.2, Nat.2 * n - Nat.2)
        Nat.1 <= Nat.2 * n - Nat.2
    }
}

/// The tree-of-edge-count claim at a vertex count of one more than `m`.
define forest_tree_shift_at[V](g: SimpleGraph[V], m: Nat) -> Bool {
    forall(s: FiniteSet[V]) {
        fs_card(s) = m + Nat.1 implies
            ((is_acyclic(g, s) and Nat.1 <= fs_card(s) and
                directed_edge_count(g, s) = Nat.2 * fs_card(s) - Nat.2)
            implies simple_graph_connected_on(g, s))
    }
}

/// A forest on `n` vertices with `n - 1` edges is a tree.
theorem forest_tree_of_edge_count[V](g: SimpleGraph[V], s: FiniteSet[V]) {
    is_acyclic(g, s) and Nat.1 <= fs_card(s) and
        directed_edge_count(g, s) = Nat.2 * fs_card(s) - Nat.2
        implies simple_graph_connected_on(g, s)
} by {
    forall(s2: FiniteSet[V]) {
        if fs_card(s2) = Nat.0 + Nat.1 {
            fs_card_one_members_eq(s2)
            forall(x: V, y: V) {
                if s2.contains(x) and s2.contains(y) {
                    x = y
                    simple_graph_reachable_refl(induced_subgraph(g, s2.underlying_set), x)
                    simple_graph_reachable(induced_subgraph(g, s2.underlying_set), x, y)
                }
                (s2.contains(x) and s2.contains(y) implies
                    simple_graph_reachable(induced_subgraph(g, s2.underlying_set), x, y))
            }
            simple_graph_connected_on(g, s2) = forall(x: V, y: V) {
                s2.contains(x) and s2.contains(y) implies
                    simple_graph_reachable(induced_subgraph(g, s2.underlying_set), x, y)
            }
            simple_graph_connected_on(g, s2)
        }
        (fs_card(s2) = Nat.0 + Nat.1 implies
            ((is_acyclic(g, s2) and Nat.1 <= fs_card(s2) and
                directed_edge_count(g, s2) = Nat.2 * fs_card(s2) - Nat.2)
            implies simple_graph_connected_on(g, s2)))
    }
    forest_tree_shift_at(g, Nat.0) = forall(s3: FiniteSet[V]) {
        fs_card(s3) = Nat.0 + Nat.1 implies
            ((is_acyclic(g, s3) and Nat.1 <= fs_card(s3) and
                directed_edge_count(g, s3) = Nat.2 * fs_card(s3) - Nat.2)
            implies simple_graph_connected_on(g, s3))
    }
    forest_tree_shift_at(g, Nat.0)
    forall(m: Nat) {
        if forest_tree_shift_at(g, m) {
            forall(s2: FiniteSet[V]) {
                if fs_card(s2) = m + Nat.1 + Nat.1 {
                    if is_acyclic(g, s2) and Nat.1 <= fs_card(s2) and
                        directed_edge_count(g, s2) = Nat.2 * fs_card(s2) - Nat.2 {
                        is_acyclic(g, s2)
                        Nat.1 <= fs_card(s2)
                        directed_edge_count(g, s2) = Nat.2 * fs_card(s2) - Nat.2
                        fs_card(s2) = m + Nat.1 + Nat.1
                        Nat.0 <= m
                        lte_add_left(Nat.2, Nat.0, m)
                        Nat.0 <= m implies Nat.2 + Nat.0 <= Nat.2 + m
                        Nat.2 + Nat.0 <= Nat.2 + m
                        Nat.2 + Nat.0 = Nat.2
                        Nat.2 <= Nat.2 + m
                        Nat.2 + m = m + Nat.2
                        Nat.2 <= m + Nat.2
                        m + Nat.2 = m + Nat.1 + Nat.1
                        Nat.2 <= m + Nat.1 + Nat.1
                        Nat.2 <= fs_card(s2)
                        two_mul_sub_two_pos(fs_card(s2))
                        Nat.1 <= Nat.2 * fs_card(s2) - Nat.2
                        Nat.1 <= directed_edge_count(g, s2)
                        forest_has_leaf(g, s2)
                        ((is_acyclic(g, s2) and Nat.1 <= directed_edge_count(g, s2)) implies
                            exists(v: V) { s2.contains(v) and is_leaf(g, s2, v) })
                        exists(v: V) { s2.contains(v) and is_leaf(g, s2, v) }
                        let v: V satisfy { s2.contains(v) and is_leaf(g, s2, v) }
                        s2.contains(v)
                        is_leaf(g, s2, v)
                        is_leaf(g, s2, v) = exists(u: V) {
                            s2.contains(u) and g.adj(v, u) and
                                forall(w: V) { s2.contains(w) and g.adj(v, w) implies w = u }
                        }
                        exists(u: V) {
                            s2.contains(u) and g.adj(v, u) and
                                forall(w: V) { s2.contains(w) and g.adj(v, w) implies w = u }
                        }
                        let u: V satisfy {
                            s2.contains(u) and g.adj(v, u) and
                                forall(w: V) { s2.contains(w) and g.adj(v, w) implies w = u }
                        }
                        s2.contains(u)
                        g.adj(v, u)
                        simple_graph_adj_ne(g, v, u)
                        v != u
                        u != v
                        edges_remove_leaf(g, s2, v)
                        ((is_leaf(g, s2, v) and s2.contains(v)) implies
                            directed_edge_count(g, s2) = directed_edge_count(g, fs_remove(s2, v)) + Nat.2)
                        directed_edge_count(g, s2) = directed_edge_count(g, fs_remove(s2, v)) + Nat.2
                        fs_card_remove_of_contains(s2, v)
                        fs_card(s2) = fs_card(fs_remove(s2, v)) + Nat.1
                        fs_card(s2) = m + Nat.1 + Nat.1
                        fs_card(fs_remove(s2, v)) + Nat.1 = m + Nat.1 + Nat.1
                        add_cancels_left(fs_card(fs_remove(s2, v)), m + Nat.1, Nat.1)
                        fs_card(fs_remove(s2, v)) = m + Nat.1
                        is_acyclic_subset(g, s2, fs_remove(s2, v))
                        fs_remove_subset(s2, v)
                        fs_remove(s2, v).subset_eq(s2)
                        is_acyclic(g, fs_remove(s2, v))
                        directed_edge_count(g, s2) = directed_edge_count(g, fs_remove(s2, v)) + Nat.2
                        directed_edge_count(g, s2) = Nat.2 * fs_card(s2) - Nat.2
                        directed_edge_count(g, fs_remove(s2, v)) + Nat.2 = Nat.2 * fs_card(s2) - Nat.2
                        fs_card(s2) = m + Nat.1 + Nat.1
                        Nat.2 * fs_card(s2) - Nat.2 = Nat.2 * (m + Nat.1 + Nat.1) - Nat.2
                        Nat.2 * (m + Nat.1 + Nat.1) = Nat.2 * (m + Nat.1) + Nat.2
                        Nat.2 * (m + Nat.1) + Nat.2 - Nat.2 = Nat.2 * (m + Nat.1)
                        add_imp_sub(Nat.2 * (m + Nat.1), Nat.2, Nat.2 * (m + Nat.1 + Nat.1))
                        Nat.2 * (m + Nat.1 + Nat.1) - Nat.2 = Nat.2 * (m + Nat.1)
                        directed_edge_count(g, fs_remove(s2, v)) + Nat.2 = Nat.2 * (m + Nat.1)
                        add_cancels_left(directed_edge_count(g, fs_remove(s2, v)),
                            Nat.2 * (m + Nat.1) - Nat.2, Nat.2)
                        directed_edge_count(g, fs_remove(s2, v)) = Nat.2 * (m + Nat.1) - Nat.2
                        fs_card(fs_remove(s2, v)) = m + Nat.1
                        directed_edge_count(g, fs_remove(s2, v)) =
                            Nat.2 * fs_card(fs_remove(s2, v)) - Nat.2
                        Nat.1 <= fs_card(fs_remove(s2, v))
                        forest_tree_shift_at(g, m) = forall(s3: FiniteSet[V]) {
                            fs_card(s3) = m + Nat.1 implies
                                ((is_acyclic(g, s3) and Nat.1 <= fs_card(s3) and
                                    directed_edge_count(g, s3) = Nat.2 * fs_card(s3) - Nat.2)
                                implies simple_graph_connected_on(g, s3))
                        }
                        forest_tree_shift_at(g, m)
                        fs_card(fs_remove(s2, v)) = m + Nat.1
                        is_acyclic(g, fs_remove(s2, v))
                        Nat.1 <= fs_card(fs_remove(s2, v))
                        directed_edge_count(g, fs_remove(s2, v)) =
                            Nat.2 * fs_card(fs_remove(s2, v)) - Nat.2
                        simple_graph_connected_on(g, fs_remove(s2, v))
                        forall(x: V, y: V) {
                            if s2.contains(x) and s2.contains(y) {
                                if x = v {
                                    if y = v {
                                        x = y
                                        simple_graph_reachable_refl(induced_subgraph(g, s2.underlying_set), x)
                                        simple_graph_reachable(induced_subgraph(g, s2.underlying_set), x, y)
                                    }
                                    if not y = v {
                                        y != v
                                        fs_remove_contains_eq(s2, v, y)
                                        fs_remove(s2, v).contains(y) = (s2.contains(y) and y != v)
                                        fs_remove(s2, v).contains(y)
                                        fs_remove_contains_eq(s2, v, u)
                                        fs_remove(s2, v).contains(u) = (s2.contains(u) and u != v)
                                        fs_remove(s2, v).contains(u)
                                        simple_graph_connected_on(g, fs_remove(s2, v)) = forall(a: V, b: V) {
                                            (fs_remove(s2, v).contains(a) and fs_remove(s2, v).contains(b) implies
                                                simple_graph_reachable(induced_subgraph(g, fs_remove(s2, v).underlying_set), a, b))
                                        }
                                        simple_graph_connected_on(g, fs_remove(s2, v))
                                        simple_graph_reachable(induced_subgraph(g, fs_remove(s2, v).underlying_set), u, y)
                                        reachable_mono_on(g, s2, fs_remove(s2, v), u, y)
                                        fs_remove_subset(s2, v)
                                        fs_remove(s2, v).subset_eq(s2)
                                        ((fs_remove(s2, v).subset_eq(s2) and
                                            simple_graph_reachable(induced_subgraph(g, fs_remove(s2, v).underlying_set), u, y)) implies
                                            simple_graph_reachable(induced_subgraph(g, s2.underlying_set), u, y))
                                        simple_graph_reachable(induced_subgraph(g, s2.underlying_set), u, y)
                                        simple_graph_reachable_of_adj(induced_subgraph(g, s2.underlying_set), v, u)
                                        induced_subgraph_adj_eq(g, s2.underlying_set, v, u)
                                        induced_subgraph(g, s2.underlying_set).adj(v, u) =
                                            (g.adj(v, u) and s2.contains(v) and s2.contains(u))
                                        induced_subgraph(g, s2.underlying_set).adj(v, u)
                                        simple_graph_reachable(induced_subgraph(g, s2.underlying_set), v, u)
                                        simple_graph_reachable_transitive(induced_subgraph(g, s2.underlying_set), v, u, y)
                                        simple_graph_reachable(induced_subgraph(g, s2.underlying_set), v, y)
                                        x = v
                                        simple_graph_reachable(induced_subgraph(g, s2.underlying_set), x, y)
                                    }
                                    simple_graph_reachable(induced_subgraph(g, s2.underlying_set), x, y)
                                }
                                if not x = v {
                                    x != v
                                    fs_remove_contains_eq(s2, v, x)
                                    fs_remove(s2, v).contains(x) = (s2.contains(x) and x != v)
                                    fs_remove(s2, v).contains(x)
                                    if y = v {
                                        fs_remove_contains_eq(s2, v, u)
                                        fs_remove(s2, v).contains(u)
                                        simple_graph_connected_on(g, fs_remove(s2, v)) = forall(a: V, b: V) {
                                            (fs_remove(s2, v).contains(a) and fs_remove(s2, v).contains(b) implies
                                                simple_graph_reachable(induced_subgraph(g, fs_remove(s2, v).underlying_set), a, b))
                                        }
                                        simple_graph_connected_on(g, fs_remove(s2, v))
                                        simple_graph_reachable(induced_subgraph(g, fs_remove(s2, v).underlying_set), x, u)
                                        reachable_mono_on(g, s2, fs_remove(s2, v), x, u)
                                        fs_remove_subset(s2, v)
                                        fs_remove(s2, v).subset_eq(s2)
                                        ((fs_remove(s2, v).subset_eq(s2) and
                                            simple_graph_reachable(induced_subgraph(g, fs_remove(s2, v).underlying_set), x, u)) implies
                                            simple_graph_reachable(induced_subgraph(g, s2.underlying_set), x, u))
                                        simple_graph_reachable(induced_subgraph(g, s2.underlying_set), x, u)
                                        simple_graph_reachable_of_adj(induced_subgraph(g, s2.underlying_set), u, v)
                                        induced_subgraph_adj_eq(g, s2.underlying_set, u, v)
                                        induced_subgraph(g, s2.underlying_set).adj(u, v) =
                                            (g.adj(u, v) and s2.contains(u) and s2.contains(v))
                                        simple_graph_adj_comm(g, v, u)
                                        g.adj(u, v)
                                        induced_subgraph(g, s2.underlying_set).adj(u, v)
                                        simple_graph_reachable(induced_subgraph(g, s2.underlying_set), u, v)
                                        simple_graph_reachable_transitive(induced_subgraph(g, s2.underlying_set), x, u, v)
                                        simple_graph_reachable(induced_subgraph(g, s2.underlying_set), x, v)
                                        y = v
                                        simple_graph_reachable(induced_subgraph(g, s2.underlying_set), x, y)
                                    }
                                    if not y = v {
                                        y != v
                                        fs_remove_contains_eq(s2, v, y)
                                        fs_remove(s2, v).contains(y)
                                        simple_graph_connected_on(g, fs_remove(s2, v)) = forall(a: V, b: V) {
                                            (fs_remove(s2, v).contains(a) and fs_remove(s2, v).contains(b) implies
                                                simple_graph_reachable(induced_subgraph(g, fs_remove(s2, v).underlying_set), a, b))
                                        }
                                        simple_graph_connected_on(g, fs_remove(s2, v))
                                        simple_graph_reachable(induced_subgraph(g, fs_remove(s2, v).underlying_set), x, y)
                                        reachable_mono_on(g, s2, fs_remove(s2, v), x, y)
                                        fs_remove_subset(s2, v)
                                        fs_remove(s2, v).subset_eq(s2)
                                        ((fs_remove(s2, v).subset_eq(s2) and
                                            simple_graph_reachable(induced_subgraph(g, fs_remove(s2, v).underlying_set), x, y)) implies
                                            simple_graph_reachable(induced_subgraph(g, s2.underlying_set), x, y))
                                        simple_graph_reachable(induced_subgraph(g, s2.underlying_set), x, y)
                                    }
                                    simple_graph_reachable(induced_subgraph(g, s2.underlying_set), x, y)
                                }
                                simple_graph_reachable(induced_subgraph(g, s2.underlying_set), x, y)
                            }
                            (s2.contains(x) and s2.contains(y) implies
                                simple_graph_reachable(induced_subgraph(g, s2.underlying_set), x, y))
                        }
                        simple_graph_connected_on(g, s2) = forall(x: V, y: V) {
                            s2.contains(x) and s2.contains(y) implies
                                simple_graph_reachable(induced_subgraph(g, s2.underlying_set), x, y)
                        }
                        simple_graph_connected_on(g, s2)
                    }
                    ((is_acyclic(g, s2) and Nat.1 <= fs_card(s2) and
                        directed_edge_count(g, s2) = Nat.2 * fs_card(s2) - Nat.2)
                        implies simple_graph_connected_on(g, s2))
                }
                (fs_card(s2) = m + Nat.1 + Nat.1 implies
                    ((is_acyclic(g, s2) and Nat.1 <= fs_card(s2) and
                        directed_edge_count(g, s2) = Nat.2 * fs_card(s2) - Nat.2)
                    implies simple_graph_connected_on(g, s2)))
            }
            forest_tree_shift_at(g, m + Nat.1) = forall(s3: FiniteSet[V]) {
                fs_card(s3) = m + Nat.1 + Nat.1 implies
                    ((is_acyclic(g, s3) and Nat.1 <= fs_card(s3) and
                        directed_edge_count(g, s3) = Nat.2 * fs_card(s3) - Nat.2)
                    implies simple_graph_connected_on(g, s3))
            }
            forest_tree_shift_at(g, m + Nat.1)
        }
    }
    forall(m: Nat) { forest_tree_shift_at(g, m) implies forest_tree_shift_at(g, m + Nat.1) }
    forall(m: Nat) { forest_tree_shift_at(g, m) implies forest_tree_shift_at(g, m.suc) }
    forest_tree_shift_at(g, Nat.0) and
        forall(m: Nat) { forest_tree_shift_at(g, m) implies forest_tree_shift_at(g, m.suc) }
    alt_induction(forest_tree_shift_at(g))
    forall(m: Nat) { forest_tree_shift_at(g, m) }
    forest_tree_shift_at(g, fs_card(s) - Nat.1)
    forest_tree_shift_at(g, fs_card(s) - Nat.1) = forall(s3: FiniteSet[V]) {
        fs_card(s3) = fs_card(s) - Nat.1 + Nat.1 implies
            ((is_acyclic(g, s3) and Nat.1 <= fs_card(s3) and
                directed_edge_count(g, s3) = Nat.2 * fs_card(s3) - Nat.2)
            implies simple_graph_connected_on(g, s3))
    }
    add_sub(fs_card(s), Nat.1)
    Nat.1 <= fs_card(s) implies fs_card(s) - Nat.1 + Nat.1 = fs_card(s)
    fs_card(s) - Nat.1 + Nat.1 = fs_card(s)
    fs_card(s) = fs_card(s)
    ((is_acyclic(g, s) and Nat.1 <= fs_card(s) and
        directed_edge_count(g, s) = Nat.2 * fs_card(s) - Nat.2)
        implies simple_graph_connected_on(g, s))
}

/// The adjacency of `g` with the new edge `x`-`y` added.
///
/// The new part is guarded by `a != b` so the relation is irreflexive even when
/// the endpoints coincide; the added edge is a loop only in that degenerate case.
define add_edge_adj[V](g: SimpleGraph[V], x: V, y: V) -> (V -> V -> Bool) {
    function(a: V, b: V) {
        g.adj(a, b) or (((a = x and b = y) or (a = y and b = x)) and a != b)
    }
}

/// The new adjacency flips on the second disjunct when the first is ruled out.
theorem add_edge_adj_symmetric_case[V](g: SimpleGraph[V], x: V, y: V, a: V, b: V) {
    add_edge_adj(g, x, y)(a, b) and not g.adj(a, b) and not (a = x and b = y)
        implies (a = y and b = x)
} by {
    if add_edge_adj(g, x, y)(a, b) and not g.adj(a, b) and not (a = x and b = y) {
        add_edge_adj(g, x, y)(a, b) = (g.adj(a, b) or (((a = x and b = y) or (a = y and b = x)) and a != b))
        g.adj(a, b) or (((a = x and b = y) or (a = y and b = x)) and a != b)
        not g.adj(a, b)
        not (a = x and b = y)
        if g.adj(a, b) {
            false
        }
        ((a = x and b = y) or (a = y and b = x)) and a != b
        (a = x and b = y) or (a = y and b = x)
        if a = x and b = y {
            false
        }
        a = y and b = x
    }
}

/// The new adjacency is symmetric.
theorem add_edge_adj_symmetric[V](g: SimpleGraph[V], x: V, y: V) {
    is_symmetric(add_edge_adj(g, x, y))
} by {
    forall(a: V, b: V) {
        if add_edge_adj(g, x, y)(a, b) {
            add_edge_adj(g, x, y)(a, b) = (g.adj(a, b) or (((a = x and b = y) or (a = y and b = x)) and a != b))
            g.adj(a, b) or (((a = x and b = y) or (a = y and b = x)) and a != b)
            simple_graph_adj_comm(g, a, b)
            if g.adj(a, b) {
                g.adj(b, a)
                g.adj(b, a) or (((b = x and a = y) or (b = y and a = x)) and b != a)
            }
            if not g.adj(a, b) {
                ((a = x and b = y) or (a = y and b = x)) and a != b
                (a = x and b = y) or (a = y and b = x)
                a != b
                if a = x and b = y {
                    a = x
                    b = y
                    b = y and a = x
                    b != a
                    (b = y and a = x) and b != a
                    ((b = x and a = y) or (b = y and a = x)) and b != a
                    g.adj(b, a) or (((b = x and a = y) or (b = y and a = x)) and b != a)
                }
                if not (a = x and b = y) {
                    add_edge_adj_symmetric_case(g, x, y, a, b)
                    (add_edge_adj(g, x, y)(a, b) and not g.adj(a, b) and not (a = x and b = y)) implies (a = y and b = x)
                    add_edge_adj(g, x, y)(a, b)
                    not g.adj(a, b)
                    not (a = x and b = y)
                    add_edge_adj(g, x, y)(a, b) and not g.adj(a, b) and not (a = x and b = y)
                    a = y and b = x
                    b = x and a = y
                    b != a
                    (b = x and a = y) and b != a
                    ((b = x and a = y) or (b = y and a = x)) and b != a
                    g.adj(b, a) or (((b = x and a = y) or (b = y and a = x)) and b != a)
                }
                g.adj(b, a) or (((b = x and a = y) or (b = y and a = x)) and b != a)
            }
            g.adj(b, a) or (((b = x and a = y) or (b = y and a = x)) and b != a)
            add_edge_adj(g, x, y)(b, a) = (g.adj(b, a) or (((b = x and a = y) or (b = y and a = x)) and b != a))
            add_edge_adj(g, x, y)(b, a)
        }
        add_edge_adj(g, x, y)(a, b) implies add_edge_adj(g, x, y)(b, a)
    }
}

/// The new adjacency is irreflexive.
theorem add_edge_adj_irreflexive[V](g: SimpleGraph[V], x: V, y: V) {
    is_irreflexive(add_edge_adj(g, x, y))
} by {
    forall(a: V) {
        if add_edge_adj(g, x, y)(a, a) {
            add_edge_adj(g, x, y)(a, a) = (g.adj(a, a) or (((a = x and a = y) or (a = y and a = x)) and a != a))
            g.adj(a, a) or (((a = x and a = y) or (a = y and a = x)) and a != a)
            simple_graph_adj_irreflexive(g, a)
            not g.adj(a, a)
            ((a = x and a = y) or (a = y and a = x)) and a != a
            a != a
            false
        }
        not add_edge_adj(g, x, y)(a, a)
    }
}

/// The graph `g` with the edge between `x` and `y` added.
///
/// When the endpoints coincide the added adjacency is empty, so the construction
/// is total; the intended use has `x != y`.
let add_edge_graph[V](g: SimpleGraph[V], x: V, y: V) -> result: SimpleGraph[V] satisfy {
    SimpleGraph.new(add_edge_adj(g, x, y)) = Option.some(result)
} by {
    add_edge_adj_symmetric(g, x, y)
    add_edge_adj_irreflexive(g, x, y)
    is_symmetric(add_edge_adj(g, x, y)) and is_irreflexive(add_edge_adj(g, x, y))
}

// Adding an edge between two vertices in different components of a forest keeps
// the graph acyclic. The statement is true: any new cycle would have to use the
// new edge, and walking the rest of it would give a path between the two
// components. The proof is not yet formalized: it needs walk-splitting machinery
// for cycles (finding the first occurrence of the new edge in the cycle walk and
// re-joining the two remaining segments into a walk between the endpoints) that
// the library does not provide yet.
// theorem add_edge_between_components_acyclic[V](g: SimpleGraph[V], s: FiniteSet[V], x: V, y: V) {
//     is_acyclic(g, s) and s.contains(x) and s.contains(y) and x != y and
//         not simple_graph_reachable(g, x, y)
//         implies is_acyclic(add_edge_graph(g, x, y), s)
// }
