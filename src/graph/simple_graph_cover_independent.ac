from nat import Nat
from finite_set import FiniteSet, fs_difference, finite_set_difference_contains_eq
from graph.simple_graph import SimpleGraph
from graph.simple_graph_independent import is_independent_in, is_independent_in_intro,
    is_independent_in_apply
from graph.simple_graph_vertex_sets import is_vertex_cover, is_vertex_cover_apply,
    is_vertex_cover_intro

numerals Nat

/// The complement of a vertex cover is an independent set.
///
/// This is the converse half of the cover-independence duality: if two vertices
/// outside the cover were adjacent, that edge would have no endpoint in the cover.
theorem independent_of_vertex_cover_complement[V](
    g: SimpleGraph[V], s: FiniteSet[V], c: FiniteSet[V]
) {
    is_vertex_cover(g, s, c) implies is_independent_in(g, fs_difference(s, c))
} by {
    if is_vertex_cover(g, s, c) {
        forall(x: V, y: V) {
            if fs_difference(s, c).contains(x) and fs_difference(s, c).contains(y) {
                finite_set_difference_contains_eq(s, c, x)
                fs_difference(s, c).contains(x) = (s.contains(x) and not c.contains(x))
                s.contains(x)
                not c.contains(x)
                finite_set_difference_contains_eq(s, c, y)
                fs_difference(s, c).contains(y) = (s.contains(y) and not c.contains(y))
                s.contains(y)
                not c.contains(y)
                if g.adj(x, y) {
                    is_vertex_cover_apply(g, s, c, x, y)
                    c.contains(x) or c.contains(y)
                    false
                }
                not g.adj(x, y)
            }
            fs_difference(s, c).contains(x) and fs_difference(s, c).contains(y) implies not g.adj(x, y)
        }
        is_independent_in_intro(g, fs_difference(s, c))
        is_independent_in(g, fs_difference(s, c))
    }
}

/// A vertex of the ambient set is either in a vertex cover or in the independent
/// complement of that cover.
theorem vertex_cover_partitions_ambient[V](
    g: SimpleGraph[V], s: FiniteSet[V], c: FiniteSet[V], v: V
) {
    s.contains(v) implies c.contains(v) or fs_difference(s, c).contains(v)
} by {
    if s.contains(v) {
        if not c.contains(v) {
            finite_set_difference_contains_eq(s, c, v)
            fs_difference(s, c).contains(v) = (s.contains(v) and not c.contains(v))
            fs_difference(s, c).contains(v)
        }
        c.contains(v) or fs_difference(s, c).contains(v)
    }
}

/// No vertex lies in both a vertex cover and its complement.
theorem vertex_cover_complement_disjoint[V](
    g: SimpleGraph[V], s: FiniteSet[V], c: FiniteSet[V], v: V
) {
    not (c.contains(v) and fs_difference(s, c).contains(v))
} by {
    if c.contains(v) and fs_difference(s, c).contains(v) {
        finite_set_difference_contains_eq(s, c, v)
        fs_difference(s, c).contains(v) = (s.contains(v) and not c.contains(v))
        not c.contains(v)
        false
    }
    not (c.contains(v) and fs_difference(s, c).contains(v))
}
