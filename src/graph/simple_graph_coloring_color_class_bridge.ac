from data.basic.logic import iff_bool, iff_bool_imp_eq
from data.basic.set import Set, set_preimage, set_preimage_contains_eq,
    set_preimage_contains_image, singleton_contains_eq
from graph.simple_graph import SimpleGraph, is_independent_set, simple_graph_adj_ne,
    is_independent_set_contains_not_adj
from graph.simple_graph_coloring import simple_graph_coloring, simple_graph_coloring_adj_ne

/// In a proper coloring, each color class is an independent set.
theorem simple_graph_coloring_color_class_independent[V, C](
    g: SimpleGraph[V],
    color: V -> C,
    c: C
) {
    simple_graph_coloring(g, color) implies
    is_independent_set(g, set_preimage(color, Set[C].singleton(c)))
} by {
    if simple_graph_coloring(g, color) {
        let color_class = set_preimage(color, Set[C].singleton(c))
        forall(x: V, y: V) {
            if color_class.contains(x) and color_class.contains(y) and x != y {
                if g.adj(x, y) {
                    simple_graph_coloring_adj_ne(g, color, x, y)
                    color(x) != color(y)

                    set_preimage_contains_eq(color, Set[C].singleton(c), x)
                    set_preimage_contains_image(color, Set[C].singleton(c), x)
                    (Set[C].singleton(c)).contains(color(x))
                    singleton_contains_eq(c, color(x))
                    c = color(x)
                    color(x) = c

                    set_preimage_contains_eq(color, Set[C].singleton(c), y)
                    set_preimage_contains_image(color, Set[C].singleton(c), y)
                    (Set[C].singleton(c)).contains(color(y))
                    singleton_contains_eq(c, color(y))
                    c = color(y)
                    color(y) = c

                    color(x) = color(y)
                    false
                }
                not g.adj(x, y)
            }
        }
        is_independent_set(g, color_class)
        is_independent_set(g, set_preimage(color, Set[C].singleton(c)))
    }
}

/// If every color class is independent, then the map is a proper coloring.
theorem simple_graph_coloring_of_color_classes_independent[V, C](
    g: SimpleGraph[V],
    color: V -> C
) {
    (forall(c: C) { is_independent_set(g, set_preimage(color, Set[C].singleton(c))) }) implies
    simple_graph_coloring(g, color)
} by {
    if forall(c: C) { is_independent_set(g, set_preimage(color, Set[C].singleton(c))) } {
        forall(x: V, y: V) {
            if g.adj(x, y) {
                simple_graph_adj_ne(g, x, y)
                x != y
                if color(x) = color(y) {
                    let color_class = set_preimage(color, Set[C].singleton(color(x)))
                    is_independent_set(g, color_class)

                    singleton_contains_eq(color(x), color(x))
                    (Set[C].singleton(color(x))).contains(color(x))
                    set_preimage_contains_eq(color, Set[C].singleton(color(x)), x)
                    set_preimage_contains_image(color, Set[C].singleton(color(x)), x)
                    color_class.contains(x)

                    singleton_contains_eq(color(x), color(y))
                    (Set[C].singleton(color(x))).contains(color(y))
                    set_preimage_contains_eq(color, Set[C].singleton(color(x)), y)
                    set_preimage_contains_image(color, Set[C].singleton(color(x)), y)
                    color_class.contains(y)

                    is_independent_set_contains_not_adj(g, color_class, x, y)
                    not g.adj(x, y)
                    false
                }
                color(x) != color(y)
            }
        }
        simple_graph_coloring(g, color)
    }
}

/// Proper colorings are exactly maps whose color classes are independent.
theorem simple_graph_coloring_iff_color_classes_independent[V, C](
    g: SimpleGraph[V],
    color: V -> C
) {
    simple_graph_coloring(g, color) =
    forall(c: C) { is_independent_set(g, set_preimage(color, Set[C].singleton(c))) }
} by {
    if simple_graph_coloring(g, color) {
        forall(c: C) {
            simple_graph_coloring_color_class_independent(g, color, c)
            is_independent_set(g, set_preimage(color, Set[C].singleton(c)))
        }
    }
    if forall(c: C) { is_independent_set(g, set_preimage(color, Set[C].singleton(c))) } {
        simple_graph_coloring_of_color_classes_independent(g, color)
        simple_graph_coloring(g, color)
    }
    iff_bool(
        simple_graph_coloring(g, color),
        forall(c: C) { is_independent_set(g, set_preimage(color, Set[C].singleton(c))) }
    )
    iff_bool_imp_eq(
        simple_graph_coloring(g, color),
        forall(c: C) { is_independent_set(g, set_preimage(color, Set[C].singleton(c))) }
    )
}
