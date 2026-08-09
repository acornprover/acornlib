from nat import Nat, lt_and_lte

numerals Nat

/// The labels of the nine vertices are ordered.
///
/// Comparisons between numerals are not free, so the ladder is stated once and read off.
theorem nine_labels_lt_ladder {
    Nat.0 < Nat.1 and Nat.1 < Nat.2 and Nat.2 < Nat.3 and Nat.3 < Nat.4 and Nat.4 < Nat.5 and Nat.5 < Nat.6 and Nat.6 < Nat.7 and Nat.7 < Nat.8 and Nat.8 < Nat.9
} by {
    Nat.0.suc = Nat.1
    Nat.0 < Nat.0.suc
    Nat.0 < Nat.1
    Nat.1.suc = Nat.2
    Nat.1 < Nat.1.suc
    Nat.1 < Nat.2
    Nat.2.suc = Nat.3
    Nat.2 < Nat.2.suc
    Nat.2 < Nat.3
    Nat.3.suc = Nat.4
    Nat.3 < Nat.3.suc
    Nat.3 < Nat.4
    Nat.4.suc = Nat.5
    Nat.4 < Nat.4.suc
    Nat.4 < Nat.5
    Nat.5.suc = Nat.6
    Nat.5 < Nat.5.suc
    Nat.5 < Nat.6
    Nat.6.suc = Nat.7
    Nat.6 < Nat.6.suc
    Nat.6 < Nat.7
    Nat.7.suc = Nat.8
    Nat.7 < Nat.7.suc
    Nat.7 < Nat.8
    Nat.8.suc = Nat.9
    Nat.8 < Nat.8.suc
    Nat.8 < Nat.9
}

/// Every label is below nine.
///
/// The flat conjunction of all nine bound facts is stated once and read off.
theorem nine_labels_lt_bound {
    Nat.0 < Nat.9 and Nat.1 < Nat.9 and Nat.2 < Nat.9 and Nat.3 < Nat.9 and Nat.4 < Nat.9 and Nat.5 < Nat.9 and Nat.6 < Nat.9 and Nat.7 < Nat.9 and Nat.8 < Nat.9
} by {
    Nat.0.suc = Nat.1
    Nat.0 < Nat.0.suc
    Nat.0 < Nat.1
    Nat.1.suc = Nat.2
    Nat.1 < Nat.1.suc
    Nat.1 < Nat.2
    Nat.2.suc = Nat.3
    Nat.2 < Nat.2.suc
    Nat.2 < Nat.3
    Nat.3.suc = Nat.4
    Nat.3 < Nat.3.suc
    Nat.3 < Nat.4
    Nat.4.suc = Nat.5
    Nat.4 < Nat.4.suc
    Nat.4 < Nat.5
    Nat.5.suc = Nat.6
    Nat.5 < Nat.5.suc
    Nat.5 < Nat.6
    Nat.6.suc = Nat.7
    Nat.6 < Nat.6.suc
    Nat.6 < Nat.7
    Nat.7.suc = Nat.8
    Nat.7 < Nat.7.suc
    Nat.7 < Nat.8
    Nat.8.suc = Nat.9
    Nat.8 < Nat.8.suc
    Nat.8 < Nat.9
    Nat.8 <= Nat.9
    lt_and_lte(Nat.7, Nat.8, Nat.9)
    Nat.7 < Nat.9
    Nat.7 <= Nat.9
    lt_and_lte(Nat.6, Nat.7, Nat.9)
    Nat.6 < Nat.9
    Nat.6 <= Nat.9
    lt_and_lte(Nat.5, Nat.6, Nat.9)
    Nat.5 < Nat.9
    Nat.5 <= Nat.9
    lt_and_lte(Nat.4, Nat.5, Nat.9)
    Nat.4 < Nat.9
    Nat.4 <= Nat.9
    lt_and_lte(Nat.3, Nat.4, Nat.9)
    Nat.3 < Nat.9
    Nat.3 <= Nat.9
    lt_and_lte(Nat.2, Nat.3, Nat.9)
    Nat.2 < Nat.9
    Nat.2 <= Nat.9
    lt_and_lte(Nat.1, Nat.2, Nat.9)
    Nat.1 < Nat.9
    Nat.1 <= Nat.9
    lt_and_lte(Nat.0, Nat.1, Nat.9)
    Nat.0 < Nat.9
}

/// The label 0 is distinct from every label above it.
theorem nine_labels_distinct_row0 {
    Nat.0 != Nat.1 and Nat.0 != Nat.2 and Nat.0 != Nat.3 and Nat.0 != Nat.4 and Nat.0 != Nat.5 and Nat.0 != Nat.6 and Nat.0 != Nat.7 and Nat.0 != Nat.8
} by {
    nine_labels_lt_ladder
    (Nat.0 < Nat.1 and Nat.1 < Nat.2 and Nat.2 < Nat.3 and Nat.3 < Nat.4 and Nat.4 < Nat.5 and Nat.5 < Nat.6 and Nat.6 < Nat.7 and Nat.7 < Nat.8 and Nat.8 < Nat.9)
    Nat.0 < Nat.1
    Nat.0 != Nat.1
    Nat.1 <= Nat.2
    lt_and_lte(Nat.0, Nat.1, Nat.2)
    Nat.0 < Nat.2
    Nat.0 != Nat.2
    Nat.2 <= Nat.3
    lt_and_lte(Nat.0, Nat.2, Nat.3)
    Nat.0 < Nat.3
    Nat.0 != Nat.3
    Nat.3 <= Nat.4
    lt_and_lte(Nat.0, Nat.3, Nat.4)
    Nat.0 < Nat.4
    Nat.0 != Nat.4
    Nat.4 <= Nat.5
    lt_and_lte(Nat.0, Nat.4, Nat.5)
    Nat.0 < Nat.5
    Nat.0 != Nat.5
    Nat.5 <= Nat.6
    lt_and_lte(Nat.0, Nat.5, Nat.6)
    Nat.0 < Nat.6
    Nat.0 != Nat.6
    Nat.6 <= Nat.7
    lt_and_lte(Nat.0, Nat.6, Nat.7)
    Nat.0 < Nat.7
    Nat.0 != Nat.7
    Nat.7 <= Nat.8
    lt_and_lte(Nat.0, Nat.7, Nat.8)
    Nat.0 < Nat.8
    Nat.0 != Nat.8
}

/// The label 1 is distinct from every label above it.
theorem nine_labels_distinct_row1 {
    Nat.1 != Nat.2 and Nat.1 != Nat.3 and Nat.1 != Nat.4 and Nat.1 != Nat.5 and Nat.1 != Nat.6 and Nat.1 != Nat.7 and Nat.1 != Nat.8
} by {
    nine_labels_lt_ladder
    (Nat.0 < Nat.1 and Nat.1 < Nat.2 and Nat.2 < Nat.3 and Nat.3 < Nat.4 and Nat.4 < Nat.5 and Nat.5 < Nat.6 and Nat.6 < Nat.7 and Nat.7 < Nat.8 and Nat.8 < Nat.9)
    Nat.1 < Nat.2
    Nat.1 != Nat.2
    Nat.2 <= Nat.3
    lt_and_lte(Nat.1, Nat.2, Nat.3)
    Nat.1 < Nat.3
    Nat.1 != Nat.3
    Nat.3 <= Nat.4
    lt_and_lte(Nat.1, Nat.3, Nat.4)
    Nat.1 < Nat.4
    Nat.1 != Nat.4
    Nat.4 <= Nat.5
    lt_and_lte(Nat.1, Nat.4, Nat.5)
    Nat.1 < Nat.5
    Nat.1 != Nat.5
    Nat.5 <= Nat.6
    lt_and_lte(Nat.1, Nat.5, Nat.6)
    Nat.1 < Nat.6
    Nat.1 != Nat.6
    Nat.6 <= Nat.7
    lt_and_lte(Nat.1, Nat.6, Nat.7)
    Nat.1 < Nat.7
    Nat.1 != Nat.7
    Nat.7 <= Nat.8
    lt_and_lte(Nat.1, Nat.7, Nat.8)
    Nat.1 < Nat.8
    Nat.1 != Nat.8
}

/// The label 2 is distinct from every label above it.
theorem nine_labels_distinct_row2 {
    Nat.2 != Nat.3 and Nat.2 != Nat.4 and Nat.2 != Nat.5 and Nat.2 != Nat.6 and Nat.2 != Nat.7 and Nat.2 != Nat.8
} by {
    nine_labels_lt_ladder
    (Nat.0 < Nat.1 and Nat.1 < Nat.2 and Nat.2 < Nat.3 and Nat.3 < Nat.4 and Nat.4 < Nat.5 and Nat.5 < Nat.6 and Nat.6 < Nat.7 and Nat.7 < Nat.8 and Nat.8 < Nat.9)
    Nat.2 < Nat.3
    Nat.2 != Nat.3
    Nat.3 <= Nat.4
    lt_and_lte(Nat.2, Nat.3, Nat.4)
    Nat.2 < Nat.4
    Nat.2 != Nat.4
    Nat.4 <= Nat.5
    lt_and_lte(Nat.2, Nat.4, Nat.5)
    Nat.2 < Nat.5
    Nat.2 != Nat.5
    Nat.5 <= Nat.6
    lt_and_lte(Nat.2, Nat.5, Nat.6)
    Nat.2 < Nat.6
    Nat.2 != Nat.6
    Nat.6 <= Nat.7
    lt_and_lte(Nat.2, Nat.6, Nat.7)
    Nat.2 < Nat.7
    Nat.2 != Nat.7
    Nat.7 <= Nat.8
    lt_and_lte(Nat.2, Nat.7, Nat.8)
    Nat.2 < Nat.8
    Nat.2 != Nat.8
}

/// The label 3 is distinct from every label above it.
theorem nine_labels_distinct_row3 {
    Nat.3 != Nat.4 and Nat.3 != Nat.5 and Nat.3 != Nat.6 and Nat.3 != Nat.7 and Nat.3 != Nat.8
} by {
    nine_labels_lt_ladder
    (Nat.0 < Nat.1 and Nat.1 < Nat.2 and Nat.2 < Nat.3 and Nat.3 < Nat.4 and Nat.4 < Nat.5 and Nat.5 < Nat.6 and Nat.6 < Nat.7 and Nat.7 < Nat.8 and Nat.8 < Nat.9)
    Nat.3 < Nat.4
    Nat.3 != Nat.4
    Nat.4 <= Nat.5
    lt_and_lte(Nat.3, Nat.4, Nat.5)
    Nat.3 < Nat.5
    Nat.3 != Nat.5
    Nat.5 <= Nat.6
    lt_and_lte(Nat.3, Nat.5, Nat.6)
    Nat.3 < Nat.6
    Nat.3 != Nat.6
    Nat.6 <= Nat.7
    lt_and_lte(Nat.3, Nat.6, Nat.7)
    Nat.3 < Nat.7
    Nat.3 != Nat.7
    Nat.7 <= Nat.8
    lt_and_lte(Nat.3, Nat.7, Nat.8)
    Nat.3 < Nat.8
    Nat.3 != Nat.8
}

/// The label 4 is distinct from every label above it.
theorem nine_labels_distinct_row4 {
    Nat.4 != Nat.5 and Nat.4 != Nat.6 and Nat.4 != Nat.7 and Nat.4 != Nat.8
} by {
    nine_labels_lt_ladder
    (Nat.0 < Nat.1 and Nat.1 < Nat.2 and Nat.2 < Nat.3 and Nat.3 < Nat.4 and Nat.4 < Nat.5 and Nat.5 < Nat.6 and Nat.6 < Nat.7 and Nat.7 < Nat.8 and Nat.8 < Nat.9)
    Nat.4 < Nat.5
    Nat.4 != Nat.5
    Nat.5 <= Nat.6
    lt_and_lte(Nat.4, Nat.5, Nat.6)
    Nat.4 < Nat.6
    Nat.4 != Nat.6
    Nat.6 <= Nat.7
    lt_and_lte(Nat.4, Nat.6, Nat.7)
    Nat.4 < Nat.7
    Nat.4 != Nat.7
    Nat.7 <= Nat.8
    lt_and_lte(Nat.4, Nat.7, Nat.8)
    Nat.4 < Nat.8
    Nat.4 != Nat.8
}

/// The label 5 is distinct from every label above it.
theorem nine_labels_distinct_row5 {
    Nat.5 != Nat.6 and Nat.5 != Nat.7 and Nat.5 != Nat.8
} by {
    nine_labels_lt_ladder
    (Nat.0 < Nat.1 and Nat.1 < Nat.2 and Nat.2 < Nat.3 and Nat.3 < Nat.4 and Nat.4 < Nat.5 and Nat.5 < Nat.6 and Nat.6 < Nat.7 and Nat.7 < Nat.8 and Nat.8 < Nat.9)
    Nat.5 < Nat.6
    Nat.5 != Nat.6
    Nat.6 <= Nat.7
    lt_and_lte(Nat.5, Nat.6, Nat.7)
    Nat.5 < Nat.7
    Nat.5 != Nat.7
    Nat.7 <= Nat.8
    lt_and_lte(Nat.5, Nat.7, Nat.8)
    Nat.5 < Nat.8
    Nat.5 != Nat.8
}

/// The label 6 is distinct from every label above it.
theorem nine_labels_distinct_row6 {
    Nat.6 != Nat.7 and Nat.6 != Nat.8
} by {
    nine_labels_lt_ladder
    (Nat.0 < Nat.1 and Nat.1 < Nat.2 and Nat.2 < Nat.3 and Nat.3 < Nat.4 and Nat.4 < Nat.5 and Nat.5 < Nat.6 and Nat.6 < Nat.7 and Nat.7 < Nat.8 and Nat.8 < Nat.9)
    Nat.6 < Nat.7
    Nat.6 != Nat.7
    Nat.7 <= Nat.8
    lt_and_lte(Nat.6, Nat.7, Nat.8)
    Nat.6 < Nat.8
    Nat.6 != Nat.8
}

/// The label 7 is distinct from every label above it.
theorem nine_labels_distinct_row7 {
    Nat.7 != Nat.8
} by {
    nine_labels_lt_ladder
    (Nat.0 < Nat.1 and Nat.1 < Nat.2 and Nat.2 < Nat.3 and Nat.3 < Nat.4 and Nat.4 < Nat.5 and Nat.5 < Nat.6 and Nat.6 < Nat.7 and Nat.7 < Nat.8 and Nat.8 < Nat.9)
    Nat.7 < Nat.8
    Nat.7 != Nat.8
}
