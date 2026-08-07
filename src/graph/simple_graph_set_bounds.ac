from nat import Nat, lte_trans
from finite_set import FiniteSet, finite_set_subset_trans
from data.finite.finite_set_card import fs_card
from data.finite.finite_set_card_compare import fs_card_mono
from graph.simple_graph import SimpleGraph
from graph.simple_graph_independent import is_independent_in, is_independent_in_of_subset,
    is_clique_in, is_clique_in_of_subset
from graph.simple_graph_vertex_sets import is_vertex_cover

numerals Nat

/// True if every independent subset of `s` has at most `n` vertices.
define independence_at_most[V](g: SimpleGraph[V], s: FiniteSet[V], n: Nat) -> Bool {
    forall(t: FiniteSet[V]) {
        t.subset_eq(s) and is_independent_in(g, t) implies fs_card(t) <= n
    }
}

/// An independent subset obeys the independence bound.
theorem independence_at_most_apply[V](
    g: SimpleGraph[V], s: FiniteSet[V], n: Nat, t: FiniteSet[V]
) {
    independence_at_most(g, s, n) and t.subset_eq(s) and is_independent_in(g, t) implies fs_card(t) <= n
} by {
    if independence_at_most(g, s, n) and t.subset_eq(s) and is_independent_in(g, t) {
        independence_at_most(g, s, n) = forall(r: FiniteSet[V]) {
            r.subset_eq(s) and is_independent_in(g, r) implies fs_card(r) <= n
        }
        forall(r: FiniteSet[V]) {
            r.subset_eq(s) and is_independent_in(g, r) implies fs_card(r) <= n
        }
        t.subset_eq(s) and is_independent_in(g, t) implies fs_card(t) <= n
        fs_card(t) <= n
    }
}

/// A pointwise bound on independent subsets is an independence bound.
theorem independence_at_most_intro[V](g: SimpleGraph[V], s: FiniteSet[V], n: Nat) {
    (forall(t: FiniteSet[V]) {
        t.subset_eq(s) and is_independent_in(g, t) implies fs_card(t) <= n
    }) implies independence_at_most(g, s, n)
} by {
    if forall(t: FiniteSet[V]) {
        t.subset_eq(s) and is_independent_in(g, t) implies fs_card(t) <= n
    } {
        independence_at_most(g, s, n) = forall(r: FiniteSet[V]) {
            r.subset_eq(s) and is_independent_in(g, r) implies fs_card(r) <= n
        }
        independence_at_most(g, s, n)
    }
}

/// Independence bounds weaken upward.
theorem independence_at_most_weaken[V](g: SimpleGraph[V], s: FiniteSet[V], m: Nat, n: Nat) {
    independence_at_most(g, s, m) and m <= n implies independence_at_most(g, s, n)
} by {
    if independence_at_most(g, s, m) and m <= n {
        forall(t: FiniteSet[V]) {
            if t.subset_eq(s) and is_independent_in(g, t) {
                independence_at_most_apply(g, s, m, t)
                fs_card(t) <= m
                lte_trans(fs_card(t), m, n)
                fs_card(t) <= n
            }
            t.subset_eq(s) and is_independent_in(g, t) implies fs_card(t) <= n
        }
        independence_at_most_intro(g, s, n)
        independence_at_most(g, s, n)
    }
}

/// The number of vertices bounds every independent subset.
theorem independence_at_most_ambient[V](g: SimpleGraph[V], s: FiniteSet[V]) {
    independence_at_most(g, s, fs_card(s))
} by {
    forall(t: FiniteSet[V]) {
        if t.subset_eq(s) and is_independent_in(g, t) {
            fs_card_mono(t, s)
            fs_card(t) <= fs_card(s)
        }
        t.subset_eq(s) and is_independent_in(g, t) implies fs_card(t) <= fs_card(s)
    }
    independence_at_most_intro(g, s, fs_card(s))
    independence_at_most(g, s, fs_card(s))
}

/// True if every clique inside `s` has at most `n` vertices.
define clique_at_most[V](g: SimpleGraph[V], s: FiniteSet[V], n: Nat) -> Bool {
    forall(t: FiniteSet[V]) {
        t.subset_eq(s) and is_clique_in(g, t) implies fs_card(t) <= n
    }
}

/// A clique obeys the clique bound.
theorem clique_at_most_apply[V](
    g: SimpleGraph[V], s: FiniteSet[V], n: Nat, t: FiniteSet[V]
) {
    clique_at_most(g, s, n) and t.subset_eq(s) and is_clique_in(g, t) implies fs_card(t) <= n
} by {
    if clique_at_most(g, s, n) and t.subset_eq(s) and is_clique_in(g, t) {
        clique_at_most(g, s, n) = forall(r: FiniteSet[V]) {
            r.subset_eq(s) and is_clique_in(g, r) implies fs_card(r) <= n
        }
        forall(r: FiniteSet[V]) {
            r.subset_eq(s) and is_clique_in(g, r) implies fs_card(r) <= n
        }
        t.subset_eq(s) and is_clique_in(g, t) implies fs_card(t) <= n
        fs_card(t) <= n
    }
}

/// A pointwise bound on cliques is a clique bound.
theorem clique_at_most_intro[V](g: SimpleGraph[V], s: FiniteSet[V], n: Nat) {
    (forall(t: FiniteSet[V]) {
        t.subset_eq(s) and is_clique_in(g, t) implies fs_card(t) <= n
    }) implies clique_at_most(g, s, n)
} by {
    if forall(t: FiniteSet[V]) {
        t.subset_eq(s) and is_clique_in(g, t) implies fs_card(t) <= n
    } {
        clique_at_most(g, s, n) = forall(r: FiniteSet[V]) {
            r.subset_eq(s) and is_clique_in(g, r) implies fs_card(r) <= n
        }
        clique_at_most(g, s, n)
    }
}

/// Clique bounds weaken upward.
theorem clique_at_most_weaken[V](g: SimpleGraph[V], s: FiniteSet[V], m: Nat, n: Nat) {
    clique_at_most(g, s, m) and m <= n implies clique_at_most(g, s, n)
} by {
    if clique_at_most(g, s, m) and m <= n {
        forall(t: FiniteSet[V]) {
            if t.subset_eq(s) and is_clique_in(g, t) {
                clique_at_most_apply(g, s, m, t)
                fs_card(t) <= m
                lte_trans(fs_card(t), m, n)
                fs_card(t) <= n
            }
            t.subset_eq(s) and is_clique_in(g, t) implies fs_card(t) <= n
        }
        clique_at_most_intro(g, s, n)
        clique_at_most(g, s, n)
    }
}

/// The number of vertices bounds every clique.
theorem clique_at_most_ambient[V](g: SimpleGraph[V], s: FiniteSet[V]) {
    clique_at_most(g, s, fs_card(s))
} by {
    forall(t: FiniteSet[V]) {
        if t.subset_eq(s) and is_clique_in(g, t) {
            fs_card_mono(t, s)
            fs_card(t) <= fs_card(s)
        }
        t.subset_eq(s) and is_clique_in(g, t) implies fs_card(t) <= fs_card(s)
    }
    clique_at_most_intro(g, s, fs_card(s))
    clique_at_most(g, s, fs_card(s))
}

/// An independence bound on a set transfers to every subset of it.
theorem independence_at_most_of_smaller[V](
    g: SimpleGraph[V], s: FiniteSet[V], r: FiniteSet[V], n: Nat
) {
    r.subset_eq(s) and independence_at_most(g, s, n) implies independence_at_most(g, r, n)
} by {
    if r.subset_eq(s) and independence_at_most(g, s, n) {
        forall(t: FiniteSet[V]) {
            if t.subset_eq(r) and is_independent_in(g, t) {
                finite_set_subset_trans(t, r, s)
                t.subset_eq(s)
                independence_at_most_apply(g, s, n, t)
                fs_card(t) <= n
            }
            t.subset_eq(r) and is_independent_in(g, t) implies fs_card(t) <= n
        }
        independence_at_most_intro(g, r, n)
        independence_at_most(g, r, n)
    }
}
