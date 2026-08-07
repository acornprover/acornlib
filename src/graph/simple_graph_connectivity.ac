from data.basic.functions import is_surjective_fn, surjective_fn_has_preimage
from data.basic.relation_basic import is_reflexive, is_symmetric, is_transitive, is_equivalence,
    relation_subset, relation_subset_step, symmetric_flip, transitive_step,
    reflexive_symmetric_transitive_imp_equivalence
from data.basic.relation_transport import relation_pullback
from data.basic.set import Set, equivalence_class, equivalence_class_contains_eq,
    equivalence_class_contains_of_related, equivalence_class_eq_of_equiv,
    equivalence_of_equivalence_class_eq, maps_into_set_image, set_ext, set_image,
    set_image_contains_witness
from data.basic.relation import relation_refl_trans_closure, relation_subset_refl_trans_closure,
    relation_refl_trans_closure_is_reflexive, relation_refl_trans_closure_is_symmetric,
    relation_refl_trans_closure_is_transitive, relation_refl_trans_closure_monotone,
    relation_refl_trans_closure_pullback_subset
from graph.simple_graph import SimpleGraph, SimpleGraphHom, SimpleGraphIso, complete_graph,
    complete_graph_adj_iff_ne, is_graph_hom, simple_graph_hom_maps_adj, simple_graph_hom_new_src,
    simple_graph_hom_new_dst, simple_graph_hom_new_map, simple_graph_iso_map_is_hom,
    simple_graph_iso_inv_is_hom, simple_graph_iso_left_inv, simple_graph_iso_right_inv

/// Vertices are reachable in a simple graph when they are related by the
/// reflexive-transitive closure of the graph adjacency relation.
define simple_graph_reachable[V](g: SimpleGraph[V], x: V, y: V) -> Bool {
    relation_refl_trans_closure(g.adj, x, y)
}

/// A simple graph is connected when every pair of vertices is reachable.
define simple_graph_connected[V](g: SimpleGraph[V]) -> Bool {
    forall(x: V, y: V) {
        simple_graph_reachable(g, x, y)
    }
}

/// Every vertex is reachable from itself.
theorem simple_graph_reachable_refl[V](g: SimpleGraph[V], x: V) {
    simple_graph_reachable(g, x, x)
} by {
    relation_refl_trans_closure_is_reflexive(g.adj)
    is_reflexive(relation_refl_trans_closure(g.adj))
    simple_graph_reachable(g, x, x)
}

/// A graph edge gives a one-step reachability witness.
theorem simple_graph_reachable_of_adj[V](g: SimpleGraph[V], x: V, y: V) {
    g.adj(x, y) implies simple_graph_reachable(g, x, y)
} by {
    relation_subset_refl_trans_closure(g.adj)
    if g.adj(x, y) {
        relation_subset_step(g.adj, relation_refl_trans_closure(g.adj), x, y)
        relation_refl_trans_closure(g.adj, x, y)
        simple_graph_reachable(g, x, y)
    }
}

/// Reachability in a simple graph is symmetric because adjacency is symmetric.
theorem simple_graph_reachable_symmetric[V](g: SimpleGraph[V], x: V, y: V) {
    simple_graph_reachable(g, x, y) implies simple_graph_reachable(g, y, x)
} by {
    relation_refl_trans_closure_is_symmetric(g.adj)
    is_symmetric(relation_refl_trans_closure(g.adj))
    if simple_graph_reachable(g, x, y) {
        relation_refl_trans_closure(g.adj, x, y)
        symmetric_flip(relation_refl_trans_closure(g.adj), x, y)
        relation_refl_trans_closure(g.adj, y, x)
        simple_graph_reachable(g, y, x)
    }
}

/// Reachability in a simple graph is transitive.
theorem simple_graph_reachable_transitive[V](g: SimpleGraph[V], x: V, y: V, z: V) {
    simple_graph_reachable(g, x, y) and simple_graph_reachable(g, y, z)
    implies simple_graph_reachable(g, x, z)
} by {
    relation_refl_trans_closure_is_transitive(g.adj)
    is_transitive(relation_refl_trans_closure(g.adj))
    if simple_graph_reachable(g, x, y) and simple_graph_reachable(g, y, z) {
        relation_refl_trans_closure(g.adj, x, y)
        relation_refl_trans_closure(g.adj, y, z)
        transitive_step(relation_refl_trans_closure(g.adj), x, y, z)
        relation_refl_trans_closure(g.adj, x, z)
        simple_graph_reachable(g, x, z)
    }
}

/// A graph homomorphism sends reachable vertices to reachable vertices.
theorem simple_graph_hom_maps_reachable[V, W](f: SimpleGraphHom[V, W], x: V, y: V) {
    simple_graph_reachable(f.src, x, y) implies
    simple_graph_reachable(f.dst, f.map(x), f.map(y))
} by {
    if simple_graph_reachable(f.src, x, y) {
        let src_adj = f.src.adj
        let pulled_adj = relation_pullback(f.map, f.dst.adj)
        forall(a: V, b: V) {
            if src_adj(a, b) {
                f.src.adj(a, b)
                simple_graph_hom_maps_adj(f, a, b)
                f.dst.adj(f.map(a), f.map(b))
                pulled_adj(a, b) = f.dst.adj(f.map(a), f.map(b))
                pulled_adj(a, b)
            }
        }
        relation_subset(src_adj, pulled_adj)
        src_adj = f.src.adj
        pulled_adj = relation_pullback(f.map, f.dst.adj)
        relation_subset(f.src.adj, relation_pullback(f.map, f.dst.adj))
        relation_refl_trans_closure_monotone(f.src.adj, relation_pullback(f.map, f.dst.adj))
        relation_subset(
            relation_refl_trans_closure(f.src.adj),
            relation_refl_trans_closure(relation_pullback(f.map, f.dst.adj))
        )
        relation_refl_trans_closure(f.src.adj, x, y)
        relation_subset_step(
            relation_refl_trans_closure(f.src.adj),
            relation_refl_trans_closure(relation_pullback(f.map, f.dst.adj)),
            x,
            y
        )
        relation_refl_trans_closure(relation_pullback(f.map, f.dst.adj), x, y)
        relation_refl_trans_closure_pullback_subset(f.map, f.dst.adj)
        relation_subset(
            relation_refl_trans_closure(relation_pullback(f.map, f.dst.adj)),
            relation_pullback(f.map, relation_refl_trans_closure(f.dst.adj))
        )
        relation_subset_step(
            relation_refl_trans_closure(relation_pullback(f.map, f.dst.adj)),
            relation_pullback(f.map, relation_refl_trans_closure(f.dst.adj)),
            x,
            y
        )
        relation_pullback(f.map, relation_refl_trans_closure(f.dst.adj), x, y)
        relation_pullback(f.map, relation_refl_trans_closure(f.dst.adj), x, y) =
            relation_refl_trans_closure(f.dst.adj, f.map(x), f.map(y))
        relation_refl_trans_closure(f.dst.adj, f.map(x), f.map(y))
        simple_graph_reachable(f.dst, f.map(x), f.map(y))
    }
}

/// A surjective graph homomorphism carries connectedness of the source to connectedness of the target.
theorem simple_graph_connected_of_surjective_hom[V, W](f: SimpleGraphHom[V, W]) {
    is_surjective_fn(f.map) and simple_graph_connected(f.src)
    implies simple_graph_connected(f.dst)
} by {
    if is_surjective_fn(f.map) and simple_graph_connected(f.src) {
        forall(u: W, v: W) {
            surjective_fn_has_preimage(f.map, u)
            let x: V satisfy {
                f.map(x) = u
            }
            surjective_fn_has_preimage(f.map, v)
            let y: V satisfy {
                f.map(y) = v
            }
            let src_conn = simple_graph_connected(f.src)
            src_conn
            src_conn = forall(a: V, b: V) {
                simple_graph_reachable(f.src, a, b)
            }
            simple_graph_reachable(f.src, x, y)
            simple_graph_hom_maps_reachable(f, x, y)
            simple_graph_reachable(f.dst, f.map(x), f.map(y))
            simple_graph_reachable(f.dst, u, v)
        }
    }
}

/// Reachability is an equivalence relation on the vertices of a simple graph.
theorem simple_graph_reachable_is_equivalence[V](g: SimpleGraph[V]) {
    is_equivalence(simple_graph_reachable(g))
} by {
    relation_refl_trans_closure_is_reflexive(g.adj)
    is_reflexive(relation_refl_trans_closure(g.adj))
    forall(x: V) {
        relation_refl_trans_closure(g.adj, x, x)
        simple_graph_reachable(g, x, x)
    }
    is_reflexive(simple_graph_reachable(g))

    relation_refl_trans_closure_is_symmetric(g.adj)
    is_symmetric(relation_refl_trans_closure(g.adj))
    forall(x: V, y: V) {
        if simple_graph_reachable(g, x, y) {
            relation_refl_trans_closure(g.adj, x, y)
            symmetric_flip(relation_refl_trans_closure(g.adj), x, y)
            relation_refl_trans_closure(g.adj, y, x)
            simple_graph_reachable(g, y, x)
        }
    }
    is_symmetric(simple_graph_reachable(g))

    relation_refl_trans_closure_is_transitive(g.adj)
    is_transitive(relation_refl_trans_closure(g.adj))
    forall(x: V, y: V, z: V) {
        if simple_graph_reachable(g, x, y) and simple_graph_reachable(g, y, z) {
            relation_refl_trans_closure(g.adj, x, y)
            relation_refl_trans_closure(g.adj, y, z)
            transitive_step(relation_refl_trans_closure(g.adj), x, y, z)
            relation_refl_trans_closure(g.adj, x, z)
            simple_graph_reachable(g, x, z)
        }
    }
    is_transitive(simple_graph_reachable(g))
    reflexive_symmetric_transitive_imp_equivalence(simple_graph_reachable(g))
}

/// The connected component of a vertex is its reachability equivalence class.
define simple_graph_component[V](g: SimpleGraph[V], x: V) -> Set[V] {
    equivalence_class(simple_graph_reachable(g), x)
}

/// Membership in a graph component is exactly reachability from its representative.
theorem simple_graph_component_contains_eq[V](g: SimpleGraph[V], x: V, y: V) {
    simple_graph_component(g, x).contains(y) = simple_graph_reachable(g, x, y)
} by {
    simple_graph_component(g, x) = equivalence_class(simple_graph_reachable(g), x)
    equivalence_class_contains_eq(simple_graph_reachable(g), x, y)
}

/// A vertex belongs to its own connected component.
theorem simple_graph_component_contains_self[V](g: SimpleGraph[V], x: V) {
    simple_graph_component(g, x).contains(x)
} by {
    simple_graph_reachable_refl(g, x)
    equivalence_class_contains_of_related(simple_graph_reachable(g), x, x)
    simple_graph_component(g, x).contains(x)
}

/// Reachable vertices determine the same connected component.
theorem simple_graph_component_eq_of_reachable[V](g: SimpleGraph[V], x: V, y: V) {
    simple_graph_reachable(g, x, y) implies simple_graph_component(g, x) = simple_graph_component(g, y)
} by {
    if simple_graph_reachable(g, x, y) {
        simple_graph_reachable_is_equivalence(g)
        equivalence_class_eq_of_equiv(simple_graph_reachable(g), x, y)
        equivalence_class(simple_graph_reachable(g), x) = equivalence_class(simple_graph_reachable(g), y)
        simple_graph_component(g, x) = simple_graph_component(g, y)
    }
}

/// Equal connected components force their representatives to be reachable.
theorem simple_graph_reachable_of_component_eq[V](g: SimpleGraph[V], x: V, y: V) {
    simple_graph_component(g, x) = simple_graph_component(g, y) implies simple_graph_reachable(g, x, y)
} by {
    if simple_graph_component(g, x) = simple_graph_component(g, y) {
        simple_graph_reachable_is_equivalence(g)
        equivalence_class(simple_graph_reachable(g), x) = equivalence_class(simple_graph_reachable(g), y)
        equivalence_of_equivalence_class_eq(simple_graph_reachable(g), x, y)
        simple_graph_reachable(g, x, y)
    }
}

/// Two connected components are equal exactly when their representatives are reachable.
theorem simple_graph_component_eq_iff_reachable[V](g: SimpleGraph[V], x: V, y: V) {
    (simple_graph_component(g, x) = simple_graph_component(g, y)) = simple_graph_reachable(g, x, y)
} by {
    if simple_graph_component(g, x) = simple_graph_component(g, y) {
        simple_graph_reachable_of_component_eq(g, x, y)
        simple_graph_reachable(g, x, y)
    }
    if simple_graph_reachable(g, x, y) {
        simple_graph_component_eq_of_reachable(g, x, y)
        simple_graph_component(g, x) = simple_graph_component(g, y)
    }
}

/// A vertex inside a connected component has the same component as the representative.
theorem simple_graph_component_eq_of_contains[V](g: SimpleGraph[V], x: V, y: V) {
    simple_graph_component(g, x).contains(y) implies simple_graph_component(g, x) = simple_graph_component(g, y)
} by {
    if simple_graph_component(g, x).contains(y) {
        simple_graph_component_contains_eq(g, x, y)
        simple_graph_reachable(g, x, y)
        simple_graph_component_eq_of_reachable(g, x, y)
        simple_graph_component(g, x) = simple_graph_component(g, y)
    }
}

/// Any two vertices in the same connected component are reachable from one another.
theorem simple_graph_component_pair_reachable[V](g: SimpleGraph[V], base: V, x: V, y: V) {
    simple_graph_component(g, base).contains(x) and simple_graph_component(g, base).contains(y)
    implies simple_graph_reachable(g, x, y)
} by {
    if simple_graph_component(g, base).contains(x) and simple_graph_component(g, base).contains(y) {
        simple_graph_component_contains_eq(g, base, x)
        simple_graph_reachable(g, base, x)
        simple_graph_reachable_symmetric(g, base, x)
        simple_graph_reachable(g, x, base)
        simple_graph_component_contains_eq(g, base, y)
        simple_graph_reachable(g, base, y)
        simple_graph_reachable_transitive(g, x, base, y)
        simple_graph_reachable(g, x, y)
    }
}

/// Components with a common vertex are equal.
theorem simple_graph_component_eq_of_common_vertex[V](g: SimpleGraph[V], x: V, y: V, z: V) {
    simple_graph_component(g, x).contains(z) and simple_graph_component(g, y).contains(z)
    implies simple_graph_component(g, x) = simple_graph_component(g, y)
} by {
    if simple_graph_component(g, x).contains(z) and simple_graph_component(g, y).contains(z) {
        simple_graph_component_contains_eq(g, x, z)
        simple_graph_reachable(g, x, z)
        simple_graph_component_contains_eq(g, y, z)
        simple_graph_reachable(g, y, z)
        simple_graph_reachable_symmetric(g, y, z)
        simple_graph_reachable(g, z, y)
        simple_graph_reachable_transitive(g, x, z, y)
        simple_graph_reachable(g, x, y)
        simple_graph_component_eq_of_reachable(g, x, y)
        simple_graph_component(g, x) = simple_graph_component(g, y)
    }
}

/// Distinct connected components are disjoint.
theorem simple_graph_component_disjoint_of_ne[V](g: SimpleGraph[V], x: V, y: V) {
    simple_graph_component(g, x) != simple_graph_component(g, y) implies
    simple_graph_component(g, x).is_disjoint(simple_graph_component(g, y))
} by {
    if simple_graph_component(g, x) != simple_graph_component(g, y) {
        forall(z: V) {
            if simple_graph_component(g, x).contains(z) and simple_graph_component(g, y).contains(z) {
                simple_graph_component_eq_of_common_vertex(g, x, y, z)
                simple_graph_component(g, x) = simple_graph_component(g, y)
                false
            }
            not (simple_graph_component(g, x).contains(z) and simple_graph_component(g, y).contains(z))
        }
    }
}

/// Any two connected components are either disjoint or equal.
theorem simple_graph_component_disjoint_or_eq[V](g: SimpleGraph[V], x: V, y: V) {
    simple_graph_component(g, x).is_disjoint(simple_graph_component(g, y)) or
    simple_graph_component(g, x) = simple_graph_component(g, y)
} by {
    if simple_graph_component(g, x) = simple_graph_component(g, y) {
        simple_graph_component(g, x).is_disjoint(simple_graph_component(g, y)) or
        simple_graph_component(g, x) = simple_graph_component(g, y)
    } else {
        simple_graph_component(g, x) != simple_graph_component(g, y)
        simple_graph_component_disjoint_of_ne(g, x, y)
        simple_graph_component(g, x).is_disjoint(simple_graph_component(g, y))
        simple_graph_component(g, x).is_disjoint(simple_graph_component(g, y)) or
        simple_graph_component(g, x) = simple_graph_component(g, y)
    }
}

/// Connected graphs have a single connected component.
theorem simple_graph_connected_component_eq[V](g: SimpleGraph[V], x: V, y: V) {
    simple_graph_connected(g) implies simple_graph_component(g, x) = simple_graph_component(g, y)
} by {
    if simple_graph_connected(g) {
        simple_graph_connected(g) = forall(a: V, b: V) {
            simple_graph_reachable(g, a, b)
        }
        simple_graph_reachable(g, x, y)
        simple_graph_component_eq_of_reachable(g, x, y)
        simple_graph_component(g, x) = simple_graph_component(g, y)
    }
}

/// A graph is connected if all vertices have the same component as one chosen vertex.
theorem simple_graph_connected_of_component_eq_base[V](g: SimpleGraph[V], base: V) {
    (forall(x: V) { simple_graph_component(g, base) = simple_graph_component(g, x) }) implies
    simple_graph_connected(g)
} by {
    if forall(x: V) { simple_graph_component(g, base) = simple_graph_component(g, x) } {
        forall(x: V, y: V) {
            simple_graph_component(g, base) = simple_graph_component(g, x)
            simple_graph_reachable_of_component_eq(g, base, x)
            simple_graph_reachable(g, base, x)
            simple_graph_reachable_symmetric(g, base, x)
            simple_graph_reachable(g, x, base)
            simple_graph_component(g, base) = simple_graph_component(g, y)
            simple_graph_reachable_of_component_eq(g, base, y)
            simple_graph_reachable(g, base, y)
            simple_graph_reachable_transitive(g, x, base, y)
            simple_graph_reachable(g, x, y)
        }
    }
}

/// A subset is connected when any two of its vertices are reachable in the ambient graph.
define simple_graph_set_connected[V](g: SimpleGraph[V], s: Set[V]) -> Bool {
    forall(x: V, y: V) {
        s.contains(x) and s.contains(y) implies simple_graph_reachable(g, x, y)
    }
}

/// Each connected component is connected as a subset of the ambient graph.
theorem simple_graph_component_set_connected[V](g: SimpleGraph[V], base: V) {
    simple_graph_set_connected(g, simple_graph_component(g, base))
} by {
    let component = simple_graph_component(g, base)
    forall(x: V, y: V) {
        if component.contains(x) and component.contains(y) {
            simple_graph_component(g, base).contains(x)
            simple_graph_component(g, base).contains(y)
            simple_graph_component_pair_reachable(g, base, x, y)
            simple_graph_reachable(g, x, y)
        }
    }
    simple_graph_set_connected(g, component)
    component = simple_graph_component(g, base)
    simple_graph_set_connected(g, simple_graph_component(g, base))
}

/// Any connected subset containing `base` is contained in the component of `base`.
theorem simple_graph_set_connected_subset_component[V](g: SimpleGraph[V], s: Set[V], base: V) {
    simple_graph_set_connected(g, s) and s.contains(base) implies
    s.subset(simple_graph_component(g, base))
} by {
    if simple_graph_set_connected(g, s) and s.contains(base) {
        simple_graph_set_connected(g, s) = forall(x: V, y: V) {
            s.contains(x) and s.contains(y) implies simple_graph_reachable(g, x, y)
        }
        forall(x: V) {
            if s.contains(x) {
                s.contains(base) and s.contains(x)
                simple_graph_reachable(g, base, x)
                simple_graph_component_contains_eq(g, base, x)
                simple_graph_component(g, base).contains(x)
            }
        }
    }
}

/// The forward map of a graph isomorphism preserves reachability.
theorem simple_graph_iso_maps_reachable[V, W](f: SimpleGraphIso[V, W], x: V, y: V) {
    simple_graph_reachable(f.src, x, y) implies
    simple_graph_reachable(f.dst, f.map(x), f.map(y))
} by {
    if simple_graph_reachable(f.src, x, y) {
        simple_graph_iso_map_is_hom(f)
        is_graph_hom(f.src, f.dst, f.map)
        let h: SimpleGraphHom[V, W] satisfy {
            SimpleGraphHom[V, W].new(f.src, f.dst, f.map) = Option.some(h)
        }
        simple_graph_hom_new_src(f.src, f.dst, f.map, h)
        h.src = f.src
        simple_graph_hom_new_dst(f.src, f.dst, f.map, h)
        h.dst = f.dst
        simple_graph_hom_new_map(f.src, f.dst, f.map, h)
        h.map = f.map
        simple_graph_hom_maps_reachable(h, x, y)
        simple_graph_reachable(h.dst, h.map(x), h.map(y))
        simple_graph_reachable(f.dst, f.map(x), f.map(y))
    }
}

/// The inverse map of a graph isomorphism reflects reachability back to the source.
theorem simple_graph_iso_reflects_reachable[V, W](f: SimpleGraphIso[V, W], x: V, y: V) {
    simple_graph_reachable(f.dst, f.map(x), f.map(y)) implies
    simple_graph_reachable(f.src, x, y)
} by {
    if simple_graph_reachable(f.dst, f.map(x), f.map(y)) {
        simple_graph_iso_inv_is_hom(f)
        is_graph_hom(f.dst, f.src, f.inv)
        let h: SimpleGraphHom[W, V] satisfy {
            SimpleGraphHom[W, V].new(f.dst, f.src, f.inv) = Option.some(h)
        }
        simple_graph_hom_new_src(f.dst, f.src, f.inv, h)
        h.src = f.dst
        simple_graph_hom_new_dst(f.dst, f.src, f.inv, h)
        h.dst = f.src
        simple_graph_hom_new_map(f.dst, f.src, f.inv, h)
        h.map = f.inv
        simple_graph_hom_maps_reachable(h, f.map(x), f.map(y))
        simple_graph_reachable(h.dst, h.map(f.map(x)), h.map(f.map(y)))
        simple_graph_iso_left_inv(f, x)
        simple_graph_iso_left_inv(f, y)
        simple_graph_reachable(f.src, x, y)
    }
}

/// Connectedness is invariant under graph isomorphism.
theorem simple_graph_iso_connected_iff[V, W](f: SimpleGraphIso[V, W]) {
    simple_graph_connected(f.src) = simple_graph_connected(f.dst)
} by {
    if simple_graph_connected(f.src) {
        forall(u: W, v: W) {
            let x = f.inv(u)
            let y = f.inv(v)
            simple_graph_connected(f.src) = forall(a: V, b: V) {
                simple_graph_reachable(f.src, a, b)
            }
            simple_graph_reachable(f.src, x, y)
            simple_graph_iso_maps_reachable(f, x, y)
            simple_graph_reachable(f.dst, f.map(x), f.map(y))
            simple_graph_iso_right_inv(f, u)
            simple_graph_iso_right_inv(f, v)
            simple_graph_reachable(f.dst, u, v)
        }
        simple_graph_connected(f.dst)
    }
    if simple_graph_connected(f.dst) {
        forall(x: V, y: V) {
            simple_graph_connected(f.dst) = forall(u: W, v: W) {
                simple_graph_reachable(f.dst, u, v)
            }
            simple_graph_reachable(f.dst, f.map(x), f.map(y))
            simple_graph_iso_reflects_reachable(f, x, y)
            simple_graph_reachable(f.src, x, y)
        }
        simple_graph_connected(f.src)
    }
}

/// The forward image of a connected component under an isomorphism lies in the corresponding target component.
theorem simple_graph_iso_maps_component_contains[V, W](f: SimpleGraphIso[V, W], x: V, y: V) {
    simple_graph_component(f.src, x).contains(y) implies
    simple_graph_component(f.dst, f.map(x)).contains(f.map(y))
} by {
    if simple_graph_component(f.src, x).contains(y) {
        simple_graph_component_contains_eq(f.src, x, y)
        simple_graph_reachable(f.src, x, y)
        simple_graph_iso_maps_reachable(f, x, y)
        simple_graph_reachable(f.dst, f.map(x), f.map(y))
        simple_graph_component_contains_eq(f.dst, f.map(x), f.map(y))
        simple_graph_component(f.dst, f.map(x)).contains(f.map(y))
    }
}

/// A graph isomorphism maps each component onto the corresponding target component.
theorem simple_graph_iso_image_component_eq[V, W](f: SimpleGraphIso[V, W], x: V) {
    set_image(simple_graph_component(f.src, x), f.map) = simple_graph_component(f.dst, f.map(x))
} by {
    let image = set_image(simple_graph_component(f.src, x), f.map)
    let target = simple_graph_component(f.dst, f.map(x))
    forall(z: W) {
        if image.contains(z) {
            set_image_contains_witness(simple_graph_component(f.src, x), f.map, z)
            let y: V satisfy {
                simple_graph_component(f.src, x).contains(y) and z = f.map(y)
            }
            simple_graph_iso_maps_component_contains(f, x, y)
            simple_graph_component(f.dst, f.map(x)).contains(f.map(y))
            f.map(y) = z
            target.contains(z)
        }
        if target.contains(z) {
            simple_graph_component_contains_eq(f.dst, f.map(x), z)
            simple_graph_reachable(f.dst, f.map(x), z)
            simple_graph_iso_right_inv(f, z)
            f.map(f.inv(z)) = z
            simple_graph_reachable(f.dst, f.map(x), f.map(f.inv(z)))
            simple_graph_iso_reflects_reachable(f, x, f.inv(z))
            simple_graph_reachable(f.src, x, f.inv(z))
            simple_graph_component_contains_eq(f.src, x, f.inv(z))
            simple_graph_component(f.src, x).contains(f.inv(z))
            z = f.map(f.inv(z))
            let y = f.inv(z)
            simple_graph_component(f.src, x).contains(y)
            maps_into_set_image(simple_graph_component(f.src, x), f.map, y)
            set_image(simple_graph_component(f.src, x), f.map).contains(f.map(y))
            z = f.map(y)
            image.contains(z)
        }
        image.contains(z) = target.contains(z)
    }
    set_ext(image, target)
}

/// Complete graphs are connected: equal endpoints use reflexivity and distinct endpoints use the edge.
theorem complete_graph_connected[V] {
    simple_graph_connected(complete_graph[V])
} by {
    let g = complete_graph[V]
    forall(x: V, y: V) {
        if x = y {
            simple_graph_reachable_refl(g, x)
            simple_graph_reachable(g, x, y)
        } else {
            x != y
            complete_graph_adj_iff_ne[V](x, y)
            g.adj(x, y)
            simple_graph_reachable_of_adj(g, x, y)
            simple_graph_reachable(g, x, y)
        }
    }
    g = complete_graph[V]
    simple_graph_connected(g)
    simple_graph_connected(complete_graph[V])
}
