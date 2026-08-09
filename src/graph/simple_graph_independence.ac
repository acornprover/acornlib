/// The independence number and the vertex cover number of a finite graph.
///
/// The classical complementarity alpha(G) + tau(G) = |V|: the complement of a vertex cover
/// is an independent set and the complement of an independent set is a vertex cover, so the
/// largest independent set and the smallest vertex cover are complementary in size. The
/// set-level duality is in `simple_graph_cover_independent.ac` and
/// `simple_graph_independent.ac`; this file draws the numerical consequences, together with
/// the coloring bound |V| <= chi(G) * alpha(G).

from nat import Nat, add_sub, add_comm, add_imp_sub, add_imp_sub_left, lte_add_left,
    lte_add_right, lte_antisymm, lt_suc_right, lt_imp_lt_suc, lt_suc, lt_or_lte,
    lt_imp_lte_suc, lte_cancel_suc, lt_not_ref, not_lt_zero
from finite_set import FiniteSet, fs_difference, fs_union, fs_insert,
    finite_set_difference_contains_eq, finite_set_union_contains_eq, finite_set_ext_contains,
    finite_set_empty_contains_eq, finite_set_singleton_subset_of_contains,
    finite_set_disjoint_union_cardinality_is, finite_set_subset_antisymm
from data.finite.finite_set_card import fs_card, fs_card_cardinality_is,
    fs_card_eq_of_cardinality_is, fs_card_singleton, fs_card_empty
from data.finite.finite_set_card_difference import fs_card_difference_of_subset
from data.finite.finite_set_subset_intro import fs_difference_subset, fs_subset_eq_intro
from data.finite.finite_set_membership import fs_insert_contains_eq
from data.finite.finite_set_filter import finite_set_filter, finite_set_filter_contains_eq,
    finite_set_filter_subset
from data.finite.finite_set_card_members import fs_two_distinct_members
from data.nat.nat_bounded_max import is_max, is_max_is_upper_bound
from data.nat.nat_range_sum import range_sum, range_sum_suc, range_sum_zero,
    range_sum_le_count_mul
from graph.simple_graph import SimpleGraph, complete_graph, complete_graph_adj_iff_ne
from graph.simple_graph_independent import is_independent_in, is_independent_in_intro,
    is_independent_in_apply, vertex_cover_of_independent_complement
from graph.simple_graph_independent_domination import is_independent_subset,
    is_independent_subset_apply, is_independent_subset_intro
from graph.simple_graph_max_independent import independence_number, independence_number_is_greatest,
    maximum_independent_set_exists, independence_number_le_card, independent_size_pred,
    independent_size_pred_intro, independence_number_attained
from graph.simple_graph_cover_number import vertex_cover_number, vertex_cover_number_is_least,
    vertex_cover_number_attained, vertex_cover_number_le_ambient, cover_size_pred
from graph.simple_graph_cover_independent import independent_of_vertex_cover_complement
from graph.simple_graph_vertex_sets import is_vertex_cover
from graph.simple_graph_chromatic import is_chromatic_number, is_chromatic_number_k_colorable,
    is_k_colorable, is_k_colorable_witness, is_proper_coloring_on, is_proper_coloring_on_apply,
    nat_range_set, nat_range_set_contains_eq, nat_range_set_card

numerals Nat

// ============================================================================
// Part 1: alpha(G) + tau(G) = |V|
// ============================================================================

/// The cover number is at most the complement of the independence number.
///
/// A maximum independent set has alpha elements, its complement is a vertex cover of size
/// |V| - alpha, and the cover number is no larger than any cover.
theorem vertex_cover_number_le_card_sub_independence_number[V](
    g: SimpleGraph[V], s: FiniteSet[V]
) {
    vertex_cover_number(g, s) <= fs_card(s) - independence_number(g, s)
} by {
    maximum_independent_set_exists(g, s)
    let (t: FiniteSet[V]) satisfy {
        is_independent_subset(g, s, t) and fs_card(t) = independence_number(g, s)
    }
    is_independent_subset_apply(g, s, t)
    t.subset_eq(s)
    is_independent_in(g, t)
    vertex_cover_of_independent_complement(g, s, t)
    is_vertex_cover(g, s, fs_difference(s, t))
    fs_difference_subset(s, t)
    fs_difference(s, t).subset_eq(s)
    vertex_cover_number_is_least(g, s, fs_difference(s, t))
    vertex_cover_number(g, s) <= fs_card(fs_difference(s, t))
    fs_card_difference_of_subset(s, t)
    fs_card(fs_difference(s, t)) = fs_card(s) - fs_card(t)
    fs_card(t) = independence_number(g, s)
    fs_card(fs_difference(s, t)) = fs_card(s) - independence_number(g, s)
    vertex_cover_number(g, s) <= fs_card(s) - independence_number(g, s)
}

/// Removing a minimum vertex cover leaves an independent set of size |V| - tau.
///
/// The complement of a cover is independent, and its size is |V| - tau, so the independence
/// number is at least |V| - tau.
theorem card_sub_cover_number_le_independence_number[V](
    g: SimpleGraph[V], s: FiniteSet[V]
) {
    fs_card(s) - vertex_cover_number(g, s) <= independence_number(g, s)
} by {
    vertex_cover_number_attained(g, s)
    cover_size_pred(g, s)(vertex_cover_number(g, s))
    cover_size_pred(g, s)(vertex_cover_number(g, s)) = exists(c: FiniteSet[V]) {
        c.subset_eq(s) and is_vertex_cover(g, s, c) and fs_card(c) = vertex_cover_number(g, s)
    }
    exists(c: FiniteSet[V]) {
        c.subset_eq(s) and is_vertex_cover(g, s, c) and fs_card(c) = vertex_cover_number(g, s)
    }
    let (c: FiniteSet[V]) satisfy {
        c.subset_eq(s) and is_vertex_cover(g, s, c) and fs_card(c) = vertex_cover_number(g, s)
    }
    independent_of_vertex_cover_complement(g, s, c)
    is_independent_in(g, fs_difference(s, c))
    fs_difference_subset(s, c)
    fs_difference(s, c).subset_eq(s)
    is_independent_subset_intro(g, s, fs_difference(s, c))
    is_independent_subset(g, s, fs_difference(s, c))
    independence_number_is_greatest(g, s, fs_difference(s, c))
    fs_card(fs_difference(s, c)) <= independence_number(g, s)
    fs_card_difference_of_subset(s, c)
    fs_card(fs_difference(s, c)) = fs_card(s) - fs_card(c)
    fs_card(c) = vertex_cover_number(g, s)
    fs_card(fs_difference(s, c)) = fs_card(s) - vertex_cover_number(g, s)
    fs_card(s) - vertex_cover_number(g, s) <= independence_number(g, s)
}

/// The sum of the two numbers is at most the number of vertices.
theorem independence_number_add_cover_number_le_card[V](
    g: SimpleGraph[V], s: FiniteSet[V]
) {
    independence_number(g, s) + vertex_cover_number(g, s) <= fs_card(s)
} by {
    vertex_cover_number_le_card_sub_independence_number(g, s)
    vertex_cover_number(g, s) <= fs_card(s) - independence_number(g, s)
    independence_number_le_card(g, s)
    independence_number(g, s) <= fs_card(s)
    add_sub(fs_card(s), independence_number(g, s))
    fs_card(s) - independence_number(g, s) + independence_number(g, s) = fs_card(s)
    lte_add_left(independence_number(g, s), vertex_cover_number(g, s),
        fs_card(s) - independence_number(g, s))
    (independence_number(g, s) + vertex_cover_number(g, s)
        <= independence_number(g, s) + (fs_card(s) - independence_number(g, s)))
    add_comm(independence_number(g, s), fs_card(s) - independence_number(g, s))
    (independence_number(g, s) + (fs_card(s) - independence_number(g, s))
        = fs_card(s) - independence_number(g, s) + independence_number(g, s))
    independence_number(g, s) + vertex_cover_number(g, s) <= fs_card(s)
}

/// The number of vertices is at most the sum of the two numbers.
theorem card_le_independence_number_add_cover_number[V](
    g: SimpleGraph[V], s: FiniteSet[V]
) {
    fs_card(s) <= independence_number(g, s) + vertex_cover_number(g, s)
} by {
    card_sub_cover_number_le_independence_number(g, s)
    fs_card(s) - vertex_cover_number(g, s) <= independence_number(g, s)
    vertex_cover_number_le_ambient(g, s)
    vertex_cover_number(g, s) <= fs_card(s)
    add_sub(fs_card(s), vertex_cover_number(g, s))
    fs_card(s) - vertex_cover_number(g, s) + vertex_cover_number(g, s) = fs_card(s)
    lte_add_right(fs_card(s) - vertex_cover_number(g, s), independence_number(g, s),
        vertex_cover_number(g, s))
    (fs_card(s) - vertex_cover_number(g, s) + vertex_cover_number(g, s)
        <= independence_number(g, s) + vertex_cover_number(g, s))
    fs_card(s) <= independence_number(g, s) + vertex_cover_number(g, s)
}

/// The classical complementarity: alpha(G) + tau(G) = |V|.
theorem independence_number_add_cover_number_eq_card[V](
    g: SimpleGraph[V], s: FiniteSet[V]
) {
    independence_number(g, s) + vertex_cover_number(g, s) = fs_card(s)
} by {
    independence_number_add_cover_number_le_card(g, s)
    independence_number(g, s) + vertex_cover_number(g, s) <= fs_card(s)
    card_le_independence_number_add_cover_number(g, s)
    fs_card(s) <= independence_number(g, s) + vertex_cover_number(g, s)
    lte_antisymm(independence_number(g, s) + vertex_cover_number(g, s), fs_card(s))
    independence_number(g, s) + vertex_cover_number(g, s) = fs_card(s)
}

/// The cover number is the complement of the independence number.
theorem vertex_cover_number_eq_card_sub_independence_number[V](
    g: SimpleGraph[V], s: FiniteSet[V]
) {
    vertex_cover_number(g, s) = fs_card(s) - independence_number(g, s)
} by {
    independence_number_add_cover_number_eq_card(g, s)
    independence_number(g, s) + vertex_cover_number(g, s) = fs_card(s)
    add_imp_sub_left(independence_number(g, s), vertex_cover_number(g, s), fs_card(s))
    fs_card(s) - independence_number(g, s) = vertex_cover_number(g, s)
    vertex_cover_number(g, s) = fs_card(s) - independence_number(g, s)
}

/// The independence number is the complement of the cover number.
theorem independence_number_eq_card_sub_cover_number[V](
    g: SimpleGraph[V], s: FiniteSet[V]
) {
    independence_number(g, s) = fs_card(s) - vertex_cover_number(g, s)
} by {
    independence_number_add_cover_number_eq_card(g, s)
    independence_number(g, s) + vertex_cover_number(g, s) = fs_card(s)
    add_imp_sub(independence_number(g, s), vertex_cover_number(g, s), fs_card(s))
    fs_card(s) - vertex_cover_number(g, s) = independence_number(g, s)
    independence_number(g, s) = fs_card(s) - vertex_cover_number(g, s)
}

// ============================================================================
// Part 2: chi(G) >= |V| / alpha(G), in the division-free form |V| <= k * alpha(G)
// ============================================================================

/// The vertices of `s` carrying the color `c`, as a finite set.
define color_class[V](s: FiniteSet[V], color: V -> Nat, c: Nat) -> FiniteSet[V] {
    finite_set_filter(s, function(v: V) { color(v) = c })
}

/// Membership in a color class is membership in `s` together with the prescribed color.
theorem color_class_contains_eq[V](s: FiniteSet[V], color: V -> Nat, c: Nat, v: V) {
    color_class(s, color, c).contains(v) = (s.contains(v) and color(v) = c)
} by {
    color_class(s, color, c) = finite_set_filter(s, function(w: V) { color(w) = c })
    color_class(s, color, c).contains(v) = finite_set_filter(s, function(w: V) { color(w) = c }).contains(v)
    finite_set_filter_contains_eq(s, function(w: V) { color(w) = c }, v)
    finite_set_filter(s, function(w: V) { color(w) = c }).contains(v) = (s.contains(v) and color(v) = c)
    color_class(s, color, c).contains(v) = (s.contains(v) and color(v) = c)
}

/// A color class is a subset of the vertex set.
theorem color_class_subset[V](s: FiniteSet[V], color: V -> Nat, c: Nat) {
    color_class(s, color, c).subset_eq(s)
} by {
    color_class(s, color, c) = finite_set_filter(s, function(w: V) { color(w) = c })
    finite_set_filter_subset(s, function(w: V) { color(w) = c })
    finite_set_filter(s, function(w: V) { color(w) = c }).subset_eq(s)
    color_class(s, color, c).subset_eq(s)
}

/// A color class of a proper coloring is an independent set.
///
/// Two vertices of the same class carry the same color, so an edge between them would violate
/// properness.
theorem color_class_independent[V](
    g: SimpleGraph[V], s: FiniteSet[V], color: V -> Nat, c: Nat
) {
    is_proper_coloring_on(g, s, color) implies is_independent_in(g, color_class(s, color, c))
} by {
    if is_proper_coloring_on(g, s, color) {
        forall(x: V, y: V) {
            if color_class(s, color, c).contains(x) and color_class(s, color, c).contains(y) {
                color_class_contains_eq(s, color, c, x)
                color_class(s, color, c).contains(x) = (s.contains(x) and color(x) = c)
                s.contains(x)
                color(x) = c
                color_class_contains_eq(s, color, c, y)
                color_class(s, color, c).contains(y) = (s.contains(y) and color(y) = c)
                s.contains(y)
                color(y) = c
                if g.adj(x, y) {
                    is_proper_coloring_on_apply(g, s, color, x, y)
                    color(x) != color(y)
                    color(x) = color(y)
                    false
                }
                not g.adj(x, y)
            }
            (color_class(s, color, c).contains(x) and color_class(s, color, c).contains(y)
                implies not g.adj(x, y))
        }
        is_independent_in_intro(g, color_class(s, color, c))
        is_independent_in(g, color_class(s, color, c))
    }
}

/// A color class of a proper coloring has size at most the independence number.
theorem color_class_card_le_independence_number[V](
    g: SimpleGraph[V], s: FiniteSet[V], color: V -> Nat, c: Nat
) {
    is_proper_coloring_on(g, s, color) implies
        fs_card(color_class(s, color, c)) <= independence_number(g, s)
} by {
    if is_proper_coloring_on(g, s, color) {
        color_class_independent(g, s, color, c)
        is_independent_in(g, color_class(s, color, c))
        color_class_subset(s, color, c)
        color_class(s, color, c).subset_eq(s)
        is_independent_subset_intro(g, s, color_class(s, color, c))
        is_independent_subset(g, s, color_class(s, color, c))
        independence_number_is_greatest(g, s, color_class(s, color, c))
        fs_card(color_class(s, color, c)) <= independence_number(g, s)
    }
}

/// The vertices of `s` whose color is below `k`, as a finite set.
define below_color[V](s: FiniteSet[V], color: V -> Nat, k: Nat) -> FiniteSet[V] {
    finite_set_filter(s, function(v: V) { color(v) < k })
}

/// Membership in a below-`k` set is membership in `s` together with a small color.
theorem below_color_contains_eq[V](s: FiniteSet[V], color: V -> Nat, k: Nat, v: V) {
    below_color(s, color, k).contains(v) = (s.contains(v) and color(v) < k)
} by {
    below_color(s, color, k) = finite_set_filter(s, function(w: V) { color(w) < k })
    below_color(s, color, k).contains(v) = finite_set_filter(s, function(w: V) { color(w) < k }).contains(v)
    finite_set_filter_contains_eq(s, function(w: V) { color(w) < k }, v)
    finite_set_filter(s, function(w: V) { color(w) < k }).contains(v) = (s.contains(v) and color(v) < k)
    below_color(s, color, k).contains(v) = (s.contains(v) and color(v) < k)
}

/// No vertex has color below zero, so the below-zero set is empty of vertices.
theorem below_color_zero_card[V](s: FiniteSet[V], color: V -> Nat) {
    fs_card(below_color(s, color, Nat.0)) = Nat.0
} by {
    forall(v: V) {
        if below_color(s, color, Nat.0).contains(v) {
            below_color_contains_eq(s, color, Nat.0, v)
            below_color(s, color, Nat.0).contains(v) = (s.contains(v) and color(v) < Nat.0)
            color(v) < Nat.0
            not_lt_zero(color(v))
            not (color(v) < Nat.0)
            false
        }
        not below_color(s, color, Nat.0).contains(v)
        below_color(s, color, Nat.0).contains(v) = false
        finite_set_empty_contains_eq(v)
        FiniteSet.empty[V].contains(v) = false
        below_color(s, color, Nat.0).contains(v) = FiniteSet.empty[V].contains(v)
        below_color(s, color, Nat.0).underlying_set.contains(v) = FiniteSet.empty[V].underlying_set.contains(v)
    }
    finite_set_ext_contains(below_color(s, color, Nat.0), FiniteSet.empty[V])
    below_color(s, color, Nat.0) = FiniteSet.empty[V]
    fs_card_empty[V]
    fs_card(FiniteSet.empty[V]) = Nat.0
    fs_card(below_color(s, color, Nat.0)) = Nat.0
}

/// A color below `k + 1` is either exactly `k` or below `k`.
///
/// The two cases split the below-`k + 1` vertices into the `k`-class and the below-`k`
/// vertices, which is the induction step for the fiber count. The union equality is proved as
/// two subset inclusions, each an implication.
theorem below_color_suc_union_subset[V](s: FiniteSet[V], color: V -> Nat, k: Nat) {
    below_color(s, color, k.suc).subset_eq(
        fs_union(below_color(s, color, k), color_class(s, color, k)))
} by {
    forall(x: V) {
        if below_color(s, color, k.suc).contains(x) {
            below_color_contains_eq(s, color, k.suc, x)
            below_color(s, color, k.suc).contains(x) = (s.contains(x) and color(x) < k.suc)
            s.contains(x)
            color(x) < k.suc
            lt_suc_right(color(x), k)
            color(x) = k or color(x) < k
            if color(x) = k {
                color_class_contains_eq(s, color, k, x)
                color_class(s, color, k).contains(x) = (s.contains(x) and color(x) = k)
                color_class(s, color, k).contains(x)
                finite_set_union_contains_eq(below_color(s, color, k), color_class(s, color, k), x)
                (fs_union(below_color(s, color, k), color_class(s, color, k)).contains(x)
                    = (below_color(s, color, k).contains(x) or color_class(s, color, k).contains(x)))
                fs_union(below_color(s, color, k), color_class(s, color, k)).contains(x)
            }
            if color(x) != k {
                if color(x) = k {
                    false
                }
                color(x) < k
                below_color_contains_eq(s, color, k, x)
                below_color(s, color, k).contains(x) = (s.contains(x) and color(x) < k)
                below_color(s, color, k).contains(x)
                finite_set_union_contains_eq(below_color(s, color, k), color_class(s, color, k), x)
                (fs_union(below_color(s, color, k), color_class(s, color, k)).contains(x)
                    = (below_color(s, color, k).contains(x) or color_class(s, color, k).contains(x)))
                fs_union(below_color(s, color, k), color_class(s, color, k)).contains(x)
            }
            fs_union(below_color(s, color, k), color_class(s, color, k)).contains(x)
        }
        (below_color(s, color, k.suc).contains(x)
            implies fs_union(below_color(s, color, k), color_class(s, color, k)).contains(x))
    }
    fs_subset_eq_intro(below_color(s, color, k.suc),
        fs_union(below_color(s, color, k), color_class(s, color, k)))
    below_color(s, color, k.suc).subset_eq(
        fs_union(below_color(s, color, k), color_class(s, color, k)))
}

/// The below-`k` vertices together with the `k`-class cover the below-`k + 1` vertices.
theorem below_color_suc_union_supset[V](s: FiniteSet[V], color: V -> Nat, k: Nat) {
    fs_union(below_color(s, color, k), color_class(s, color, k)).subset_eq(
        below_color(s, color, k.suc))
} by {
    forall(x: V) {
        if fs_union(below_color(s, color, k), color_class(s, color, k)).contains(x) {
            finite_set_union_contains_eq(below_color(s, color, k), color_class(s, color, k), x)
            (fs_union(below_color(s, color, k), color_class(s, color, k)).contains(x)
                = (below_color(s, color, k).contains(x) or color_class(s, color, k).contains(x)))
            below_color(s, color, k).contains(x) or color_class(s, color, k).contains(x)
            if below_color(s, color, k).contains(x) {
                below_color_contains_eq(s, color, k, x)
                below_color(s, color, k).contains(x) = (s.contains(x) and color(x) < k)
                s.contains(x)
                color(x) < k
                lt_imp_lt_suc(color(x), k)
                color(x) < k.suc
                below_color_contains_eq(s, color, k.suc, x)
                below_color(s, color, k.suc).contains(x) = (s.contains(x) and color(x) < k.suc)
                below_color(s, color, k.suc).contains(x)
            }
            if not below_color(s, color, k).contains(x) {
                color_class(s, color, k).contains(x)
                color_class_contains_eq(s, color, k, x)
                color_class(s, color, k).contains(x) = (s.contains(x) and color(x) = k)
                s.contains(x)
                color(x) = k
                lt_suc(k)
                k < k.suc
                color(x) < k.suc
                below_color_contains_eq(s, color, k.suc, x)
                below_color(s, color, k.suc).contains(x) = (s.contains(x) and color(x) < k.suc)
                below_color(s, color, k.suc).contains(x)
            }
            below_color(s, color, k.suc).contains(x)
        }
        (fs_union(below_color(s, color, k), color_class(s, color, k)).contains(x)
            implies below_color(s, color, k.suc).contains(x))
    }
    fs_subset_eq_intro(fs_union(below_color(s, color, k), color_class(s, color, k)),
        below_color(s, color, k.suc))
    fs_union(below_color(s, color, k), color_class(s, color, k)).subset_eq(
        below_color(s, color, k.suc))
}

/// The below-`k + 1` vertices are exactly the below-`k` vertices together with the `k`-class.
theorem below_color_suc_union[V](s: FiniteSet[V], color: V -> Nat, k: Nat) {
    below_color(s, color, k.suc) = fs_union(below_color(s, color, k), color_class(s, color, k))
} by {
    below_color_suc_union_subset(s, color, k)
    below_color(s, color, k.suc).subset_eq(
        fs_union(below_color(s, color, k), color_class(s, color, k)))
    below_color_suc_union_supset(s, color, k)
    fs_union(below_color(s, color, k), color_class(s, color, k)).subset_eq(
        below_color(s, color, k.suc))
    finite_set_subset_antisymm(below_color(s, color, k.suc),
        fs_union(below_color(s, color, k), color_class(s, color, k)))
    below_color(s, color, k.suc) = fs_union(below_color(s, color, k), color_class(s, color, k))
}

/// The below-`k` vertices and the `k`-class have no vertex in common.
theorem below_color_disjoint_class[V](s: FiniteSet[V], color: V -> Nat, k: Nat) {
    below_color(s, color, k).is_disjoint(color_class(s, color, k))
} by {
    forall(x: V) {
        if below_color(s, color, k).underlying_set.contains(x)
            and color_class(s, color, k).underlying_set.contains(x) {
            below_color_contains_eq(s, color, k, x)
            below_color(s, color, k).contains(x) = (s.contains(x) and color(x) < k)
            color(x) < k
            color_class_contains_eq(s, color, k, x)
            color_class(s, color, k).contains(x) = (s.contains(x) and color(x) = k)
            color(x) = k
            k < k
            lt_not_ref(k)
            not (k < k)
            false
        }
    }
    below_color(s, color, k).underlying_set.is_disjoint(color_class(s, color, k).underlying_set)
    below_color(s, color, k).is_disjoint(color_class(s, color, k))
}

/// The below-`k + 1` vertices split into the below-`k` vertices and the `k`-class.
///
/// The two parts are disjoint, so their sizes add.
theorem below_color_suc_split[V](s: FiniteSet[V], color: V -> Nat, k: Nat) {
    (fs_card(below_color(s, color, k.suc))
        = fs_card(below_color(s, color, k)) + fs_card(color_class(s, color, k)))
} by {
    below_color_suc_union(s, color, k)
    below_color(s, color, k.suc) = fs_union(below_color(s, color, k), color_class(s, color, k))
    fs_card_cardinality_is(below_color(s, color, k))
    below_color(s, color, k).cardinality_is(fs_card(below_color(s, color, k)))
    fs_card_cardinality_is(color_class(s, color, k))
    color_class(s, color, k).cardinality_is(fs_card(color_class(s, color, k)))
    below_color_disjoint_class(s, color, k)
    below_color(s, color, k).is_disjoint(color_class(s, color, k))
    finite_set_disjoint_union_cardinality_is(below_color(s, color, k), color_class(s, color, k),
        fs_card(below_color(s, color, k)), fs_card(color_class(s, color, k)))
    fs_union(below_color(s, color, k), color_class(s, color, k)).cardinality_is(
        fs_card(below_color(s, color, k)) + fs_card(color_class(s, color, k)))
    below_color(s, color, k.suc).cardinality_is(
        fs_card(below_color(s, color, k)) + fs_card(color_class(s, color, k)))
    fs_card_eq_of_cardinality_is(below_color(s, color, k.suc),
        fs_card(below_color(s, color, k)) + fs_card(color_class(s, color, k)))
    (fs_card(below_color(s, color, k.suc))
        = fs_card(below_color(s, color, k)) + fs_card(color_class(s, color, k)))
}

/// The sum of the first `k` class sizes counts the vertices with color below `k`.
///
/// Each vertex with color below `k` lies in exactly one of the classes indexed below `k`, so
/// induction on `k` using the split above carries the sum.
theorem range_sum_color_class_eq_below[V](s: FiniteSet[V], color: V -> Nat, k: Nat) {
    (range_sum(function(c: Nat) { fs_card(color_class(s, color, c)) }, k)
        = fs_card(below_color(s, color, k)))
} by {
    define p(x: Nat) -> Bool {
        (range_sum(function(c: Nat) { fs_card(color_class(s, color, c)) }, x)
            = fs_card(below_color(s, color, x)))
    }
    range_sum_zero(function(c: Nat) { fs_card(color_class(s, color, c)) })
    range_sum(function(c: Nat) { fs_card(color_class(s, color, c)) }, Nat.0) = Nat.0
    below_color_zero_card(s, color)
    fs_card(below_color(s, color, Nat.0)) = Nat.0
    p(Nat.0)
    forall(kk: Nat) {
        if p(kk) {
            (range_sum(function(c: Nat) { fs_card(color_class(s, color, c)) }, kk)
                = fs_card(below_color(s, color, kk)))
            range_sum_suc(function(c: Nat) { fs_card(color_class(s, color, c)) }, kk)
            (range_sum(function(c: Nat) { fs_card(color_class(s, color, c)) }, kk.suc)
                = range_sum(function(c: Nat) { fs_card(color_class(s, color, c)) }, kk)
                    + fs_card(color_class(s, color, kk)))
            (range_sum(function(c: Nat) { fs_card(color_class(s, color, c)) }, kk.suc)
                = fs_card(below_color(s, color, kk)) + fs_card(color_class(s, color, kk)))
            below_color_suc_split(s, color, kk)
            (fs_card(below_color(s, color, kk.suc))
                = fs_card(below_color(s, color, kk)) + fs_card(color_class(s, color, kk)))
            (range_sum(function(c: Nat) { fs_card(color_class(s, color, c)) }, kk.suc)
                = fs_card(below_color(s, color, kk.suc)))
            p(kk.suc)
        }
        (p(kk) implies p(kk.suc))
    }
    p(Nat.0) and forall(kk: Nat) { p(kk) implies p(kk.suc) }
    Nat.induction(p)
    p(k)
    p(k) = (range_sum(function(c: Nat) { fs_card(color_class(s, color, c)) }, k)
        = fs_card(below_color(s, color, k)))
}

/// When every vertex of `s` has color below `k`, the below-`k` set is all of `s`.
theorem below_color_card_eq_card[V](s: FiniteSet[V], color: V -> Nat, k: Nat) {
    ((forall(v: V) { s.contains(v) implies color(v) < k })
        implies fs_card(below_color(s, color, k)) = fs_card(s))
} by {
    if forall(v: V) { s.contains(v) implies color(v) < k } {
        forall(v: V) {
            below_color_contains_eq(s, color, k, v)
            below_color(s, color, k).contains(v) = (s.contains(v) and color(v) < k)
            if s.contains(v) {
                color(v) < k
                below_color(s, color, k).contains(v) = (s.contains(v) and color(v) < k)
                (s.contains(v) and color(v) < k) = true
                below_color(s, color, k).contains(v) = true
                s.contains(v) = true
                below_color(s, color, k).contains(v) = s.contains(v)
            }
            if not s.contains(v) {
                below_color(s, color, k).contains(v) = (s.contains(v) and color(v) < k)
                (s.contains(v) and color(v) < k) = false
                below_color(s, color, k).contains(v) = false
                s.contains(v) = false
                below_color(s, color, k).contains(v) = s.contains(v)
            }
            below_color(s, color, k).contains(v) = s.contains(v)
            below_color(s, color, k).underlying_set.contains(v) = s.underlying_set.contains(v)
        }
        finite_set_ext_contains(below_color(s, color, k), s)
        below_color(s, color, k) = s
        fs_card(below_color(s, color, k)) = fs_card(s)
    }
}

/// A `k`-coloring forces |V| <= k * alpha(G).
///
/// The `k` color classes are independent, so each has at most alpha vertices, and they
/// partition the vertices, so the vertex count is a sum of `k` terms each bounded by alpha.
theorem k_colorable_card_le_mul_independence_number[V](
    g: SimpleGraph[V], s: FiniteSet[V], k: Nat
) {
    is_k_colorable(g, s, k) implies fs_card(s) <= k * independence_number(g, s)
} by {
    if is_k_colorable(g, s, k) {
        is_k_colorable_witness(g, s, k)
        let (color: V -> Nat) satisfy {
            is_proper_coloring_on(g, s, color) and forall(v: V) {
                s.contains(v) implies color(v) < k
            }
        }
        is_proper_coloring_on(g, s, color)
        forall(v: V) { s.contains(v) implies color(v) < k }
        forall(c: Nat) {
            color_class_card_le_independence_number(g, s, color, c)
            fs_card(color_class(s, color, c)) <= independence_number(g, s)
        }
        range_sum_le_count_mul(function(c: Nat) { fs_card(color_class(s, color, c)) },
            independence_number(g, s), k)
        (range_sum(function(c: Nat) { fs_card(color_class(s, color, c)) }, k)
            <= k * independence_number(g, s))
        range_sum_color_class_eq_below(s, color, k)
        (range_sum(function(c: Nat) { fs_card(color_class(s, color, c)) }, k)
            = fs_card(below_color(s, color, k)))
        fs_card(below_color(s, color, k)) <= k * independence_number(g, s)
        below_color_card_eq_card(s, color, k)
        fs_card(below_color(s, color, k)) = fs_card(s)
        fs_card(s) <= k * independence_number(g, s)
    }
}

/// The chromatic number satisfies chi(G) * alpha(G) >= |V|.
///
/// Since the chromatic number is a colorability, the bound follows from the colorability
/// version. The division-free form |V| <= k * alpha(G) is the classical inequality
/// chi(G) >= |V| / alpha(G) with the division moved to the other side.
theorem chromatic_number_card_le_mul_independence_number[V](
    g: SimpleGraph[V], s: FiniteSet[V], k: Nat
) {
    is_chromatic_number(g, s, k) implies fs_card(s) <= k * independence_number(g, s)
} by {
    if is_chromatic_number(g, s, k) {
        is_chromatic_number_k_colorable(g, s, k)
        is_k_colorable(g, s, k)
        k_colorable_card_le_mul_independence_number(g, s, k)
        fs_card(s) <= k * independence_number(g, s)
    }
}

// ============================================================================
// Part 3: Small cases. The complete graph K_n: alpha = 1, tau = n - 1.
// ============================================================================

/// An independent set of a complete graph has at most one vertex.
///
/// Any two members of an independent set would be adjacent in the complete graph, so a set of
/// size at least two cannot be independent.
theorem complete_graph_independent_card_le_one[V](t: FiniteSet[V]) {
    is_independent_in(complete_graph[V], t) implies fs_card(t) <= Nat.1
} by {
    if is_independent_in(complete_graph[V], t) {
        if Nat.2 <= fs_card(t) {
            fs_two_distinct_members(t)
            let (a: V, b: V) satisfy {
                t.contains(a) and t.contains(b) and a != b
            }
            complete_graph_adj_iff_ne[V](a, b)
            complete_graph[V].adj(a, b) = (a != b)
            complete_graph[V].adj(a, b)
            is_independent_in_apply(complete_graph[V], t, a, b)
            not complete_graph[V].adj(a, b)
            false
        }
        not (Nat.2 <= fs_card(t))
        lt_or_lte(fs_card(t), Nat.2)
        fs_card(t) < Nat.2 or Nat.2 <= fs_card(t)
        fs_card(t) < Nat.2
        lt_imp_lte_suc(fs_card(t), Nat.2)
        fs_card(t).suc <= Nat.2
        lte_cancel_suc(fs_card(t), Nat.1)
        fs_card(t) <= Nat.1
    }
}

/// A singleton is independent in the complete graph.
///
/// A singleton has no two distinct members, so the adjacency condition is vacuous except for
/// the loop, which the complete graph lacks.
theorem complete_graph_singleton_independent[V](x: V) {
    is_independent_in(complete_graph[V], FiniteSet.empty[V].insert(x))
} by {
    forall(a: V, b: V) {
        if FiniteSet.empty[V].insert(x).contains(a) and FiniteSet.empty[V].insert(x).contains(b) {
            fs_insert_contains_eq(FiniteSet.empty[V], x, a)
            FiniteSet.empty[V].insert(x).contains(a) = (a = x or FiniteSet.empty[V].contains(a))
            finite_set_empty_contains_eq(a)
            FiniteSet.empty[V].contains(a) = false
            a = x
            fs_insert_contains_eq(FiniteSet.empty[V], x, b)
            FiniteSet.empty[V].insert(x).contains(b) = (b = x or FiniteSet.empty[V].contains(b))
            finite_set_empty_contains_eq(b)
            FiniteSet.empty[V].contains(b) = false
            b = x
            a = b
            complete_graph_adj_iff_ne[V](a, b)
            complete_graph[V].adj(a, b) = (a != b)
            not complete_graph[V].adj(a, b)
        }
        (FiniteSet.empty[V].insert(x).contains(a) and FiniteSet.empty[V].insert(x).contains(b)
            implies not complete_graph[V].adj(a, b))
    }
    is_independent_in_intro(complete_graph[V], FiniteSet.empty[V].insert(x))
    is_independent_in(complete_graph[V], FiniteSet.empty[V].insert(x))
}

/// The independence number of the complete graph on `n` vertices is one.
///
/// The singleton `{0}` is independent of size one, and no larger independent set exists.
theorem complete_graph_nat_range_independence_number(n: Nat) {
    Nat.0 < n implies independence_number(complete_graph[Nat], nat_range_set(n)) = Nat.1
} by {
    if Nat.0 < n {
        nat_range_set_contains_eq(n, Nat.0)
        nat_range_set(n).contains(Nat.0) = (Nat.0 < n)
        nat_range_set(n).contains(Nat.0)
        finite_set_singleton_subset_of_contains(nat_range_set(n), Nat.0)
        FiniteSet.empty[Nat].insert(Nat.0).subset_eq(nat_range_set(n))
        complete_graph_singleton_independent[Nat](Nat.0)
        is_independent_in(complete_graph[Nat], FiniteSet.empty[Nat].insert(Nat.0))
        is_independent_subset_intro(complete_graph[Nat], nat_range_set(n),
            FiniteSet.empty[Nat].insert(Nat.0))
        is_independent_subset(complete_graph[Nat], nat_range_set(n),
            FiniteSet.empty[Nat].insert(Nat.0))
        independent_size_pred_intro(complete_graph[Nat], nat_range_set(n),
            FiniteSet.empty[Nat].insert(Nat.0))
        independent_size_pred(complete_graph[Nat], nat_range_set(n))(
            fs_card(FiniteSet.empty[Nat].insert(Nat.0)))
        fs_card_singleton(Nat.0)
        fs_card(FiniteSet.empty[Nat].insert(Nat.0)) = Nat.1
        independent_size_pred(complete_graph[Nat], nat_range_set(n))(Nat.1)
        is_max(independent_size_pred(complete_graph[Nat], nat_range_set(n)),
            independence_number(complete_graph[Nat], nat_range_set(n)))
        is_max_is_upper_bound(independent_size_pred(complete_graph[Nat], nat_range_set(n)),
            independence_number(complete_graph[Nat], nat_range_set(n)), Nat.1)
        Nat.1 <= independence_number(complete_graph[Nat], nat_range_set(n))
        independence_number_attained(complete_graph[Nat], nat_range_set(n))
        independent_size_pred(complete_graph[Nat], nat_range_set(n))(
            independence_number(complete_graph[Nat], nat_range_set(n)))
        independent_size_pred(complete_graph[Nat], nat_range_set(n))(
            independence_number(complete_graph[Nat], nat_range_set(n))) = exists(t: FiniteSet[Nat]) {
                is_independent_subset(complete_graph[Nat], nat_range_set(n), t)
                    and fs_card(t) = independence_number(complete_graph[Nat], nat_range_set(n))
            }
        exists(t: FiniteSet[Nat]) {
            is_independent_subset(complete_graph[Nat], nat_range_set(n), t)
                and fs_card(t) = independence_number(complete_graph[Nat], nat_range_set(n))
        }
        let (t: FiniteSet[Nat]) satisfy {
            is_independent_subset(complete_graph[Nat], nat_range_set(n), t)
                and fs_card(t) = independence_number(complete_graph[Nat], nat_range_set(n))
        }
        is_independent_subset_apply(complete_graph[Nat], nat_range_set(n), t)
        t.subset_eq(nat_range_set(n))
        is_independent_in(complete_graph[Nat], t)
        complete_graph_independent_card_le_one(t)
        fs_card(t) <= Nat.1
        fs_card(t) = independence_number(complete_graph[Nat], nat_range_set(n))
        independence_number(complete_graph[Nat], nat_range_set(n)) <= Nat.1
        lte_antisymm(Nat.1, independence_number(complete_graph[Nat], nat_range_set(n)))
        Nat.1 = independence_number(complete_graph[Nat], nat_range_set(n))
        independence_number(complete_graph[Nat], nat_range_set(n)) = Nat.1
    }
}

/// The vertex cover number of the complete graph on `n` vertices is `n - 1`.
///
/// By the complementarity theorem, tau = |V| - alpha = n - 1.
theorem complete_graph_nat_range_cover_number(n: Nat) {
    Nat.0 < n implies vertex_cover_number(complete_graph[Nat], nat_range_set(n)) = n - Nat.1
} by {
    if Nat.0 < n {
        vertex_cover_number_eq_card_sub_independence_number(complete_graph[Nat], nat_range_set(n))
        (vertex_cover_number(complete_graph[Nat], nat_range_set(n))
            = fs_card(nat_range_set(n))
                - independence_number(complete_graph[Nat], nat_range_set(n)))
        nat_range_set_card(n)
        fs_card(nat_range_set(n)) = n
        complete_graph_nat_range_independence_number(n)
        independence_number(complete_graph[Nat], nat_range_set(n)) = Nat.1
        vertex_cover_number(complete_graph[Nat], nat_range_set(n)) = n - Nat.1
    }
}

/// The complementarity relation alpha + tau = |V| holds for the complete graph.
theorem complete_graph_nat_range_independence_add_cover(n: Nat) {
    Nat.0 < n implies
        (independence_number(complete_graph[Nat], nat_range_set(n))
            + vertex_cover_number(complete_graph[Nat], nat_range_set(n)) = n)
} by {
    if Nat.0 < n {
        independence_number_add_cover_number_eq_card(complete_graph[Nat], nat_range_set(n))
        (independence_number(complete_graph[Nat], nat_range_set(n))
            + vertex_cover_number(complete_graph[Nat], nat_range_set(n))
            = fs_card(nat_range_set(n)))
        nat_range_set_card(n)
        fs_card(nat_range_set(n)) = n
        (independence_number(complete_graph[Nat], nat_range_set(n))
            + vertex_cover_number(complete_graph[Nat], nat_range_set(n)) = n)
    }
}

// Paths and cycles remain open: alpha(P_n) = ceil(n / 2) and tau(P_n) = floor(n / 2), with the
// even and odd cycles analogous, need a rounding-free formulation of ceil and floor before the
// counting arguments (even vertices in a path, alternating vertices in a cycle) can be
// formalized. The complete graph above already demonstrates the pattern: prove the
// independence number, then read the cover number off the complementarity theorem.
