from nat import Nat
from finite_set import FiniteSet
from graph.simple_graph import SimpleGraph
from graph.simple_graph_zero_forcing_rule import is_forcing_closed, derived_set
from graph.simple_graph_zero_forcing_closure import forcing_iterate, forcing_iterate_zero,
    forcing_iterate_suc, forcing_iterate_stationary

numerals Nat

/// A colouring that is already closed never changes, however long the iteration runs.
theorem forcing_iterate_constant_of_closed[V](
    g: SimpleGraph[V], s: FiniteSet[V], b: FiniteSet[V], n: Nat
) {
    b.subset_eq(s) and is_forcing_closed(g, s, b)
        implies forcing_iterate(g, s, b, n) = b
} by {
    if b.subset_eq(s) and is_forcing_closed(g, s, b) {
        define p(x: Nat) -> Bool {
            forcing_iterate(g, s, b, x) = b
        }
        forcing_iterate_zero(g, s, b)
        forcing_iterate(g, s, b, Nat.0) = b
        p(Nat.0)
        forall(k: Nat) {
            if p(k) {
                forcing_iterate(g, s, b, k) = b
                forcing_iterate_stationary(g, s, b, Nat.0, Nat.0)
                forcing_iterate(g, s, b, Nat.0.suc) = forcing_iterate(g, s, b, Nat.0)
                forcing_iterate(g, s, b, Nat.0.suc) = b
                forcing_iterate_suc(g, s, b, k)
                forcing_iterate(g, s, b, k.suc) = derived_set(g, s, forcing_iterate(g, s, b, k))
                forcing_iterate(g, s, b, k.suc) = derived_set(g, s, b)
                forcing_iterate_suc(g, s, b, Nat.0)
                (forcing_iterate(g, s, b, Nat.0.suc)
                    = derived_set(g, s, forcing_iterate(g, s, b, Nat.0)))
                forcing_iterate(g, s, b, Nat.0.suc) = derived_set(g, s, b)
                derived_set(g, s, b) = b
                forcing_iterate(g, s, b, k.suc) = b
                p(k.suc)
            }
            (p(k) implies p(k.suc))
        }
        p(Nat.0) and forall(k: Nat) {
            p(k) implies p(k.suc)
        }
        Nat.induction(p)
        p(n)
        forcing_iterate(g, s, b, n) = b
    }
}
