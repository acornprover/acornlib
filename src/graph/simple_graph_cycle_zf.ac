from nat import Nat, lt_and_lte, lte_and_lt, lte_trans, lte_suc_suc, lte_antisymm, small_mod, divides_self, div_imp_mod
from finite_set import FiniteSet, finite_set_subset_antisymm, finite_set_subset_trans,
    finite_set_subset_contains
from data.finite.finite_set_card import fs_card
from data.finite.finite_set_card_members import fs_card_two_of_distinct_members
from data.nat.nat_range_set import range_set, range_set_contains, range_set_contains_eq, range_set_lt,
    range_set_subset, range_set_card
from data.finite.finite_set_subset_intro import fs_subset_eq_intro
from graph.simple_graph import SimpleGraph
from graph.simple_graph_cycle import cycle_graph, cycle_graph_adj_iff, cycle_graph_adj_suc
from graph.simple_graph_zero_forcing_rule import is_white_neighbor, can_force, can_force_apply,
    can_force_intro, is_forceable, is_forceable_intro, is_forceable_witness, derived_set,
    derived_set_contains_eq, is_forcing_closed
from graph.simple_graph_zero_forcing_closure import forcing_iterate, forcing_iterate_zero,
    forcing_iterate_suc, forcing_iterate_subset, is_zero_forcing_set,
    is_zero_forcing_set_intro, is_zero_forcing_set_apply
from graph.simple_graph_zero_forcing_closed import forcing_iterate_constant_of_closed
from graph.simple_graph_zero_forcing_lower import has_two_white_neighbors,
    has_two_white_neighbors_intro, two_white_neighbors_blocks_forcing,
    zero_forcing_set_nonempty
from graph.simple_graph_zero_forcing_mono import derived_set_mono
from graph.simple_graph_zero_forcing_number import zero_forcing_number, zero_forcing_number_is_least

numerals Nat

/// Inside the range, the only forward neighbour of a vertex is the next one.
///
/// The hypothesis `1 <= k` is what rules out the wrap-around edge: at `k = 0` the vertex `n - 1`
/// is also a neighbour, which is exactly why one seed does not suffice on a cycle.
theorem cycle_later_neighbor_is_suc(n: Nat, k: Nat, w: Nat) {
    Nat.1 <= k and k.suc < n and w < n and k.suc <= w and cycle_graph(n).adj(k, w)
        implies w = k.suc
} by {
    if Nat.1 <= k and k.suc < n and w < n and k.suc <= w and cycle_graph(n).adj(k, w) {
        cycle_graph_adj_iff(n, k, w)
        (cycle_graph(n).adj(k, w) = (k != w and (k.suc.mod(n) = w or w.suc.mod(n) = k)))
        (k.suc.mod(n) = w or w.suc.mod(n) = k)
        if k.suc.mod(n) = w {
            small_mod(k.suc, n)
            k.suc.mod(n) = k.suc
            w = k.suc
        }
        if k.suc.mod(n) != w {
            w.suc.mod(n) = k
            w.suc <= n
            (w.suc < n or w.suc = n)
            if w.suc < n {
                small_mod(w.suc, n)
                w.suc.mod(n) = w.suc
                w.suc = k
                w < w.suc
                w < k
                k < k.suc
                lt_and_lte(w, k, k.suc)
                w < k.suc
                lte_and_lt(k.suc, w, k.suc)
                k.suc < k.suc
                false
            }
            w.suc = n
            divides_self(n)
            div_imp_mod(n, n)
            n.mod(n) = Nat.0
            w.suc.mod(n) = Nat.0
            k = Nat.0
            Nat.1 <= Nat.0
            Nat.0 < Nat.1
            lte_and_lt(Nat.1, Nat.0, Nat.1)
            Nat.1 < Nat.1
            false
        }
        w = k.suc
    }
}

/// The last blue vertex of a prefix of the cycle forces the next one.
theorem cycle_prefix_can_force(n: Nat, k: Nat) {
    Nat.1 <= k and k.suc < n
        implies can_force(cycle_graph(n), range_set(n), range_set(k.suc), k, k.suc)
} by {
    if Nat.1 <= k and k.suc < n {
        k < k.suc
        range_set_contains(k.suc, k)
        range_set(k.suc).contains(k)
        range_set_contains(n, k.suc)
        range_set(n).contains(k.suc)
        range_set_contains_eq(k.suc, k.suc)
        range_set(k.suc).contains(k.suc) = (k.suc < k.suc)
        not range_set(k.suc).contains(k.suc)
        cycle_graph_adj_suc(n, k)
        cycle_graph(n).adj(k, k.suc)
        (is_white_neighbor(cycle_graph(n), range_set(n), range_set(k.suc), k, k.suc)
            = (range_set(n).contains(k.suc) and not range_set(k.suc).contains(k.suc)
                and cycle_graph(n).adj(k, k.suc)))
        is_white_neighbor(cycle_graph(n), range_set(n), range_set(k.suc), k, k.suc)
        forall(w: Nat) {
            if is_white_neighbor(cycle_graph(n), range_set(n), range_set(k.suc), k, w) {
                (is_white_neighbor(cycle_graph(n), range_set(n), range_set(k.suc), k, w)
                    = (range_set(n).contains(w) and not range_set(k.suc).contains(w)
                        and cycle_graph(n).adj(k, w)))
                range_set(n).contains(w)
                range_set_lt(n, w)
                w < n
                not range_set(k.suc).contains(w)
                range_set_contains_eq(k.suc, w)
                range_set(k.suc).contains(w) = (w < k.suc)
                not (w < k.suc)
                k.suc <= w
                cycle_graph(n).adj(k, w)
                cycle_later_neighbor_is_suc(n, k, w)
                w = k.suc
            }
            (is_white_neighbor(cycle_graph(n), range_set(n), range_set(k.suc), k, w)
                implies w = k.suc)
        }
        can_force_intro(cycle_graph(n), range_set(n), range_set(k.suc), k, k.suc)
        can_force(cycle_graph(n), range_set(n), range_set(k.suc), k, k.suc)
    }
}

/// One round of forcing from a blue prefix of the cycle reaches the next vertex.
///
/// Only a containment, not an equality: on a cycle the vertex `0` also has a single white
/// neighbour, namely `n - 1` across the wrap-around edge, so the blue set grows at both ends
/// at once. The containment is all that reaching the whole cycle needs, and it is what
/// composes with `derived_set_mono`.
theorem cycle_derived_set_step(n: Nat, k: Nat) {
    Nat.1 <= k and k.suc < n implies
        range_set(k.suc.suc).subset_eq(
            derived_set(cycle_graph(n), range_set(n), range_set(k.suc)))
} by {
    if Nat.1 <= k and k.suc < n {
        forall(v: Nat) {
            if range_set(k.suc.suc).contains(v) {
                range_set_lt(k.suc.suc, v)
                v < k.suc.suc
                v <= k.suc
                (v < k.suc or v = k.suc)
                if v < k.suc {
                    range_set_contains(k.suc, v)
                    range_set(k.suc).contains(v)
                    lt_and_lte(v, k.suc, n)
                    v < n
                    range_set_contains(n, v)
                    range_set(n).contains(v)
                    derived_set_contains_eq(cycle_graph(n), range_set(n), range_set(k.suc), v)
                    derived_set(cycle_graph(n), range_set(n), range_set(k.suc)).contains(v)
                }
                if v = k.suc {
                    cycle_prefix_can_force(n, k)
                    can_force(cycle_graph(n), range_set(n), range_set(k.suc), k, k.suc)
                    is_forceable_intro(cycle_graph(n), range_set(n), range_set(k.suc), k, k.suc)
                    is_forceable(cycle_graph(n), range_set(n), range_set(k.suc), k.suc)
                    is_forceable(cycle_graph(n), range_set(n), range_set(k.suc), v)
                    range_set_contains(n, k.suc)
                    range_set(n).contains(v)
                    derived_set_contains_eq(cycle_graph(n), range_set(n), range_set(k.suc), v)
                    derived_set(cycle_graph(n), range_set(n), range_set(k.suc)).contains(v)
                }
                derived_set(cycle_graph(n), range_set(n), range_set(k.suc)).contains(v)
            }
            (range_set(k.suc.suc).contains(v)
                implies derived_set(cycle_graph(n), range_set(n), range_set(k.suc)).contains(v))
        }
        fs_subset_eq_intro(range_set(k.suc.suc),
            derived_set(cycle_graph(n), range_set(n), range_set(k.suc)))
        range_set(k.suc.suc).subset_eq(
            derived_set(cycle_graph(n), range_set(n), range_set(k.suc)))
    }
}

/// After `k` rounds the blue set covers at least the prefix of length `k + 2`.
///
/// A containment rather than an equality, because the wrap-around edge lets the blue set grow
/// backwards from `0` as well; `derived_set_mono` carries the containment through each round.
theorem cycle_forcing_iterate_prefix(n: Nat, k: Nat) {
    k.suc.suc <= n implies
        range_set(k.suc.suc).subset_eq(
            forcing_iterate(cycle_graph(n), range_set(n), range_set(Nat.2), k))
} by {
    define p(x: Nat) -> Bool {
        x.suc.suc <= n implies
            range_set(x.suc.suc).subset_eq(
                forcing_iterate(cycle_graph(n), range_set(n), range_set(Nat.2), x))
    }
    forcing_iterate_zero(cycle_graph(n), range_set(n), range_set(Nat.2))
    (forcing_iterate(cycle_graph(n), range_set(n), range_set(Nat.2), Nat.0)
        = range_set(Nat.2))
    Nat.0.suc.suc = Nat.2
    range_set(Nat.2).subset_eq(range_set(Nat.2))
    p(Nat.0)
    forall(m: Nat) {
        if p(m) {
            if m.suc.suc.suc <= n {
                m.suc.suc <= m.suc.suc.suc
                lte_trans(m.suc.suc, m.suc.suc.suc, n)
                m.suc.suc <= n
                (m.suc.suc <= n implies
                    range_set(m.suc.suc).subset_eq(
                        forcing_iterate(cycle_graph(n), range_set(n), range_set(Nat.2), m)))
                range_set(m.suc.suc).subset_eq(
                    forcing_iterate(cycle_graph(n), range_set(n), range_set(Nat.2), m))
                Nat.0 <= m
                lte_suc_suc(Nat.0, m)
                Nat.0.suc <= m.suc
                lte_suc_suc(Nat.0.suc, m.suc)
                Nat.0.suc.suc <= m.suc.suc
                Nat.0.suc.suc = Nat.2
                Nat.2 <= m.suc.suc
                lte_trans(Nat.2, m.suc.suc, n)
                Nat.2 <= n
                range_set_subset(Nat.2, n)
                range_set(Nat.2).subset_eq(range_set(n))
                forcing_iterate_subset(cycle_graph(n), range_set(n), range_set(Nat.2), m)
                forcing_iterate(cycle_graph(n), range_set(n), range_set(Nat.2), m).subset_eq(
                    range_set(n))
                derived_set_mono(cycle_graph(n), range_set(n), range_set(m.suc.suc),
                    forcing_iterate(cycle_graph(n), range_set(n), range_set(Nat.2), m))
                derived_set(cycle_graph(n), range_set(n), range_set(m.suc.suc)).subset_eq(
                    derived_set(cycle_graph(n), range_set(n),
                        forcing_iterate(cycle_graph(n), range_set(n), range_set(Nat.2), m)))
                forcing_iterate_suc(cycle_graph(n), range_set(n), range_set(Nat.2), m)
                (forcing_iterate(cycle_graph(n), range_set(n), range_set(Nat.2), m.suc)
                    = derived_set(cycle_graph(n), range_set(n),
                        forcing_iterate(cycle_graph(n), range_set(n), range_set(Nat.2), m)))
                derived_set(cycle_graph(n), range_set(n), range_set(m.suc.suc)).subset_eq(
                    forcing_iterate(cycle_graph(n), range_set(n), range_set(Nat.2), m.suc))
                Nat.1 <= m.suc
                m.suc.suc < m.suc.suc.suc
                lt_and_lte(m.suc.suc, m.suc.suc.suc, n)
                m.suc.suc < n
                cycle_derived_set_step(n, m.suc)
                range_set(m.suc.suc.suc).subset_eq(
                    derived_set(cycle_graph(n), range_set(n), range_set(m.suc.suc)))
                finite_set_subset_trans(range_set(m.suc.suc.suc),
                    derived_set(cycle_graph(n), range_set(n), range_set(m.suc.suc)),
                    forcing_iterate(cycle_graph(n), range_set(n), range_set(Nat.2), m.suc))
                range_set(m.suc.suc.suc).subset_eq(
                    forcing_iterate(cycle_graph(n), range_set(n), range_set(Nat.2), m.suc))
            }
            p(m.suc)
        }
        (p(m) implies p(m.suc))
    }
    p(Nat.0) and forall(m: Nat) {
        p(m) implies p(m.suc)
    }
    Nat.induction(p)
    p(k)
}

/// Two adjacent vertices are a zero forcing set of any cycle.
///
/// After `n - 1` rounds the blue set contains every vertex, and it never leaves the vertex set,
/// so the two coincide.
theorem cycle_pair_is_zero_forcing(n: Nat) {
    Nat.2 <= n implies
        is_zero_forcing_set(cycle_graph(n.suc), range_set(n.suc), range_set(Nat.2))
} by {
    if Nat.2 <= n {
        n <= n.suc
        lte_trans(Nat.2, n, n.suc)
        Nat.2 <= n.suc
        range_set_subset(Nat.2, n.suc)
        range_set(Nat.2).subset_eq(range_set(n.suc))
        Nat.1 <= n
        (n - Nat.1).suc.suc = n.suc
        n.suc <= n.suc
        (n - Nat.1).suc.suc <= n.suc
        cycle_forcing_iterate_prefix(n.suc, n - Nat.1)
        range_set((n - Nat.1).suc.suc).subset_eq(
            forcing_iterate(cycle_graph(n.suc), range_set(n.suc), range_set(Nat.2), n - Nat.1))
        range_set(n.suc).subset_eq(
            forcing_iterate(cycle_graph(n.suc), range_set(n.suc), range_set(Nat.2), n - Nat.1))
        forcing_iterate_subset(cycle_graph(n.suc), range_set(n.suc), range_set(Nat.2),
            n - Nat.1)
        forcing_iterate(cycle_graph(n.suc), range_set(n.suc), range_set(Nat.2),
            n - Nat.1).subset_eq(range_set(n.suc))
        finite_set_subset_antisymm(
            forcing_iterate(cycle_graph(n.suc), range_set(n.suc), range_set(Nat.2), n - Nat.1),
            range_set(n.suc))
        (forcing_iterate(cycle_graph(n.suc), range_set(n.suc), range_set(Nat.2), n - Nat.1)
            = range_set(n.suc))
        is_zero_forcing_set_intro(cycle_graph(n.suc), range_set(n.suc), range_set(Nat.2),
            n - Nat.1)
        is_zero_forcing_set(cycle_graph(n.suc), range_set(n.suc), range_set(Nat.2))
    }
}

/// The zero forcing number of a cycle is at most two.
///
/// The classical value is exactly two. Two seeds are genuinely needed: a single blue vertex has
/// two white neighbours on a cycle and so forces nothing.
theorem cycle_zero_forcing_number_le_two(n: Nat) {
    Nat.2 <= n implies zero_forcing_number(cycle_graph(n.suc), range_set(n.suc)) <= Nat.2
} by {
    if Nat.2 <= n {
        cycle_pair_is_zero_forcing(n)
        is_zero_forcing_set(cycle_graph(n.suc), range_set(n.suc), range_set(Nat.2))
        zero_forcing_number_is_least(cycle_graph(n.suc), range_set(n.suc), range_set(Nat.2))
        (zero_forcing_number(cycle_graph(n.suc), range_set(n.suc))
            <= fs_card(range_set(Nat.2)))
        range_set_card(Nat.2)
        fs_card(range_set(Nat.2)) = Nat.2
        zero_forcing_number(cycle_graph(n.suc), range_set(n.suc)) <= Nat.2
    }
}

/// True if `w1` and `w2` are two distinct in-range neighbours of `v`, neither of them `v`.
///
/// Named because the existential below has seven conjuncts written out, which proof search
/// will not instantiate; as a single predicate it is a two-variable statement.
define is_neighbor_pair(n: Nat, v: Nat, w1: Nat, w2: Nat) -> Bool {
    w1 != w2 and w1 != v and w2 != v and w1 < n and w2 < n
        and cycle_graph(n).adj(v, w1) and cycle_graph(n).adj(v, w2)
}

/// The parts assemble into the pair predicate.
theorem is_neighbor_pair_intro(n: Nat, v: Nat, w1: Nat, w2: Nat) {
    w1 != w2 and w1 != v and w2 != v and w1 < n and w2 < n
        and cycle_graph(n).adj(v, w1) and cycle_graph(n).adj(v, w2)
        implies is_neighbor_pair(n, v, w1, w2)
} by {
    if w1 != w2 and w1 != v and w2 != v and w1 < n and w2 < n
        and cycle_graph(n).adj(v, w1) and cycle_graph(n).adj(v, w2) {
        (is_neighbor_pair(n, v, w1, w2)
            = (w1 != w2 and w1 != v and w2 != v and w1 < n and w2 < n
                and cycle_graph(n).adj(v, w1) and cycle_graph(n).adj(v, w2)))
        is_neighbor_pair(n, v, w1, w2)
    }
}

/// The pair predicate yields two distinct white-eligible neighbours.
theorem is_neighbor_pair_apply(n: Nat, v: Nat, w1: Nat, w2: Nat) {
    is_neighbor_pair(n, v, w1, w2) implies
        (w1 != w2 and w1 != v and w2 != v and w1 < n and w2 < n
            and cycle_graph(n).adj(v, w1) and cycle_graph(n).adj(v, w2))
} by {
    if is_neighbor_pair(n, v, w1, w2) {
        (is_neighbor_pair(n, v, w1, w2)
            = (w1 != w2 and w1 != v and w2 != v and w1 < n and w2 < n
                and cycle_graph(n).adj(v, w1) and cycle_graph(n).adj(v, w2)))
    }
}

/// Every vertex of a cycle of length at least three has two distinct neighbours in range.
///
/// The successor and the predecessor, both taken modulo the length. The length is written
/// `p + 1` and the vertex is matched as zero or a successor, so that neither the predecessor of
/// a vertex nor the last vertex needs truncated subtraction to name.
///
/// Three cases, because at `0` the predecessor wraps to the last vertex and at the last vertex
/// the successor wraps to `0`. The two neighbours coincide only when the cycle is shorter than
/// three, which is what the hypothesis rules out.
theorem cycle_two_neighbors(p: Nat, v: Nat) {
    Nat.2 <= p and v < p.suc implies exists(w1: Nat, w2: Nat) {
        is_neighbor_pair(p.suc, v, w1, w2)
    }
} by {
    if Nat.2 <= p and v < p.suc {
        divides_self(p.suc)
        div_imp_mod(p.suc, p.suc)
        p.suc.mod(p.suc) = Nat.0
        p < p.suc
        Nat.0 < Nat.2
        lt_and_lte(Nat.0, Nat.2, p)
        Nat.0 < p
        Nat.1 < Nat.2
        lt_and_lte(Nat.1, Nat.2, p)
        Nat.1 < p
        Nat.1 < p.suc
        small_mod(Nat.1, p.suc)
        Nat.1.mod(p.suc) = Nat.1
        match v {
            Nat.zero {
                v = Nat.0
                Nat.0.suc = Nat.1
                Nat.0.suc.mod(p.suc) = Nat.1
                Nat.0 != Nat.1
                cycle_graph_adj_iff(p.suc, Nat.0, Nat.1)
                (cycle_graph(p.suc).adj(Nat.0, Nat.1)
                    = (Nat.0 != Nat.1
                        and (Nat.0.suc.mod(p.suc) = Nat.1 or Nat.1.suc.mod(p.suc) = Nat.0)))
                cycle_graph(p.suc).adj(Nat.0, Nat.1)
                Nat.0 != p
                cycle_graph_adj_iff(p.suc, Nat.0, p)
                (cycle_graph(p.suc).adj(Nat.0, p)
                    = (Nat.0 != p
                        and (Nat.0.suc.mod(p.suc) = p or p.suc.mod(p.suc) = Nat.0)))
                cycle_graph(p.suc).adj(Nat.0, p)
                Nat.1 != p
                is_neighbor_pair_intro(p.suc, Nat.0, Nat.1, p)
                is_neighbor_pair(p.suc, Nat.0, Nat.1, p)
                exists(x: Nat, y: Nat) { is_neighbor_pair(p.suc, Nat.0, x, y) }
                exists(x: Nat, y: Nat) { is_neighbor_pair(p.suc, v, x, y) }
            }
            Nat.suc(u) {
                v = u.suc
                u.suc < p.suc
                u < u.suc
                u.suc <= p.suc
                lt_and_lte(u, u.suc, p.suc)
                u < p.suc
                small_mod(u.suc, p.suc)
                u.suc.mod(p.suc) = u.suc
                u.suc != u
                cycle_graph_adj_iff(p.suc, u.suc, u)
                (cycle_graph(p.suc).adj(u.suc, u)
                    = (u.suc != u
                        and (u.suc.suc.mod(p.suc) = u or u.suc.mod(p.suc) = u.suc)))
                cycle_graph(p.suc).adj(u.suc, u)
                if u.suc.suc < p.suc {
                    small_mod(u.suc.suc, p.suc)
                    u.suc.suc.mod(p.suc) = u.suc.suc
                    u.suc != u.suc.suc
                    cycle_graph_adj_iff(p.suc, u.suc, u.suc.suc)
                    (cycle_graph(p.suc).adj(u.suc, u.suc.suc)
                        = (u.suc != u.suc.suc
                            and (u.suc.suc.mod(p.suc) = u.suc.suc
                                or u.suc.suc.suc.mod(p.suc) = u.suc)))
                    cycle_graph(p.suc).adj(u.suc, u.suc.suc)
                    u < u.suc
                    u.suc < u.suc.suc
                    lt_and_lte(u, u.suc, u.suc.suc)
                    u != u.suc.suc
                    is_neighbor_pair_intro(p.suc, u.suc, u, u.suc.suc)
                    is_neighbor_pair(p.suc, u.suc, u, u.suc.suc)
                    exists(x: Nat, y: Nat) { is_neighbor_pair(p.suc, u.suc, x, y) }
                }
                if not (u.suc.suc < p.suc) {
                    p.suc <= u.suc.suc
                    u.suc < p.suc
                    u.suc.suc <= p.suc
                    lte_antisymm(u.suc.suc, p.suc)
                    u.suc.suc = p.suc
                    u.suc.suc.mod(p.suc) = Nat.0
                    Nat.0 < u.suc
                    u.suc != Nat.0
                    cycle_graph_adj_iff(p.suc, u.suc, Nat.0)
                    (cycle_graph(p.suc).adj(u.suc, Nat.0)
                        = (u.suc != Nat.0
                            and (u.suc.suc.mod(p.suc) = Nat.0
                                or Nat.0.suc.mod(p.suc) = u.suc)))
                    cycle_graph(p.suc).adj(u.suc, Nat.0)
                    u.suc = p
                    Nat.1 < u.suc
                    Nat.0 < Nat.1
                    lt_and_lte(Nat.0, Nat.1, u)
                    Nat.0 != u
                    is_neighbor_pair_intro(p.suc, u.suc, u, Nat.0)
                    is_neighbor_pair(p.suc, u.suc, u, Nat.0)
                    exists(x: Nat, y: Nat) { is_neighbor_pair(p.suc, u.suc, x, y) }
                }
                exists(x: Nat, y: Nat) { is_neighbor_pair(p.suc, u.suc, x, y) }
                exists(x: Nat, y: Nat) { is_neighbor_pair(p.suc, v, x, y) }
            }
        }
    }
}

/// A single blue vertex on a cycle forces nothing.
///
/// It has two distinct neighbours, both in range and both different from it, hence both white;
/// the uniqueness clause of the rule then fails.
theorem cycle_singleton_blocks_forcing(p: Nat, b: FiniteSet[Nat], v: Nat) {
    Nat.2 <= p and v < p.suc and (forall(x: Nat) { b.contains(x) implies x = v })
        implies is_forcing_closed(cycle_graph(p.suc), range_set(p.suc), b)
} by {
    if Nat.2 <= p and v < p.suc and forall(x: Nat) { b.contains(x) implies x = v } {
        cycle_two_neighbors(p, v)
        exists(w1: Nat, w2: Nat) { is_neighbor_pair(p.suc, v, w1, w2) }
        let (w1: Nat, w2: Nat) satisfy {
            is_neighbor_pair(p.suc, v, w1, w2)
        }
        is_neighbor_pair_apply(p.suc, v, w1, w2)
        (w1 != w2 and w1 != v and w2 != v and w1 < p.suc and w2 < p.suc
            and cycle_graph(p.suc).adj(v, w1) and cycle_graph(p.suc).adj(v, w2))
        range_set_contains(p.suc, w1)
        range_set(p.suc).contains(w1)
        range_set_contains(p.suc, w2)
        range_set(p.suc).contains(w2)
        if b.contains(w1) {
            (b.contains(w1) implies w1 = v)
            w1 = v
            false
        }
        not b.contains(w1)
        if b.contains(w2) {
            (b.contains(w2) implies w2 = v)
            w2 = v
            false
        }
        not b.contains(w2)
        forall(u: Nat) {
            if b.contains(u) {
                (b.contains(u) implies u = v)
                u = v
                (is_white_neighbor(cycle_graph(p.suc), range_set(p.suc), b, u, w1)
                    = (range_set(p.suc).contains(w1) and not b.contains(w1)
                        and cycle_graph(p.suc).adj(u, w1)))
                is_white_neighbor(cycle_graph(p.suc), range_set(p.suc), b, u, w1)
                (is_white_neighbor(cycle_graph(p.suc), range_set(p.suc), b, u, w2)
                    = (range_set(p.suc).contains(w2) and not b.contains(w2)
                        and cycle_graph(p.suc).adj(u, w2)))
                is_white_neighbor(cycle_graph(p.suc), range_set(p.suc), b, u, w2)
                has_two_white_neighbors_intro(cycle_graph(p.suc), range_set(p.suc), b, u,
                    w1, w2)
                has_two_white_neighbors(cycle_graph(p.suc), range_set(p.suc), b, u)
            }
            (b.contains(u) implies
                has_two_white_neighbors(cycle_graph(p.suc), range_set(p.suc), b, u))
        }
        two_white_neighbors_blocks_forcing(cycle_graph(p.suc), range_set(p.suc), b)
        is_forcing_closed(cycle_graph(p.suc), range_set(p.suc), b)
    }
}

/// A zero forcing set of a cycle has at least two vertices.
///
/// The matching lower bound: with one blue vertex the colouring is closed from the start, so it
/// never reaches the whole cycle. Together with the construction above, the zero forcing number
/// of a cycle of length at least three is exactly two.
theorem cycle_zero_forcing_set_card(p: Nat, b: FiniteSet[Nat]) {
    Nat.2 <= p and b.subset_eq(range_set(p.suc))
        and is_zero_forcing_set(cycle_graph(p.suc), range_set(p.suc), b)
        implies Nat.2 <= fs_card(b)
} by {
    if Nat.2 <= p and b.subset_eq(range_set(p.suc))
        and is_zero_forcing_set(cycle_graph(p.suc), range_set(p.suc), b) {
        Nat.0 < p.suc
        range_set_contains(p.suc, Nat.0)
        range_set(p.suc).contains(Nat.0)
        zero_forcing_set_nonempty(cycle_graph(p.suc), range_set(p.suc), b, Nat.0)
        exists(y: Nat) { b.contains(y) }
        let (v: Nat) satisfy {
            b.contains(v)
        }
        finite_set_subset_contains(b, range_set(p.suc), v)
        range_set(p.suc).contains(v)
        range_set_lt(p.suc, v)
        v < p.suc
        if fs_card(b) < Nat.2 {
            forall(x: Nat) {
                if b.contains(x) {
                    if x != v {
                        fs_card_two_of_distinct_members(b, x, v)
                        Nat.2 <= fs_card(b)
                        lte_and_lt(Nat.2, fs_card(b), Nat.2)
                        Nat.2 < Nat.2
                        false
                    }
                    x = v
                }
                (b.contains(x) implies x = v)
            }
            cycle_singleton_blocks_forcing(p, b, v)
            is_forcing_closed(cycle_graph(p.suc), range_set(p.suc), b)
            is_zero_forcing_set_apply(cycle_graph(p.suc), range_set(p.suc), b)
            exists(k: Nat) { forcing_iterate(cycle_graph(p.suc), range_set(p.suc), b, k)
                = range_set(p.suc) }
            let (k: Nat) satisfy {
                forcing_iterate(cycle_graph(p.suc), range_set(p.suc), b, k) = range_set(p.suc)
            }
            forcing_iterate_constant_of_closed(cycle_graph(p.suc), range_set(p.suc), b, k)
            forcing_iterate(cycle_graph(p.suc), range_set(p.suc), b, k) = b
            b = range_set(p.suc)
            range_set_card(p.suc)
            fs_card(range_set(p.suc)) = p.suc
            fs_card(b) = p.suc
            Nat.2 <= p
            p < p.suc
            lte_and_lt(Nat.2, p, p.suc)
            Nat.2 < p.suc
            Nat.2 < fs_card(b)
            lte_and_lt(Nat.2, fs_card(b), Nat.2)
            Nat.2 < Nat.2
            false
        }
        not (fs_card(b) < Nat.2)
        Nat.2 <= fs_card(b)
    }
}
