from nat import Nat
from finite_set import FiniteSet, finite_set_subset_contains
from data.finite.finite_set_card import fs_card
from data.finite.finite_set_membership import fs_insert_contains_eq, fs_insert_contains_self,
    fs_insert_contains_of_contains, fs_insert_contains_cases
from graph.simple_graph import SimpleGraph, simple_graph_adj_comm, simple_graph_adj_irreflexive
from graph.simple_graph_independent import is_independent_in, is_independent_in_apply,
    is_independent_in_intro
from graph.simple_graph_domination import is_dominating_set, is_dominating_set_intro,
    is_dominated, is_dominated_of_contains, is_dominated_of_adj
from graph.simple_graph_domination_number import domination_number, domination_number_is_least

numerals Nat

/// True if `t` is an independent subset of the ambient set `s`.
define is_independent_subset[V](
    g: SimpleGraph[V], s: FiniteSet[V], t: FiniteSet[V]
) -> Bool {
    t.subset_eq(s) and is_independent_in(g, t)
}

/// An independent subset is a subset that is independent.
theorem is_independent_subset_apply[V](
    g: SimpleGraph[V], s: FiniteSet[V], t: FiniteSet[V]
) {
    is_independent_subset(g, s, t) implies t.subset_eq(s) and is_independent_in(g, t)
} by {
    if is_independent_subset(g, s, t) {
        is_independent_subset(g, s, t) = (t.subset_eq(s) and is_independent_in(g, t))
        t.subset_eq(s) and is_independent_in(g, t)
    }
}

/// A subset that is independent is an independent subset.
theorem is_independent_subset_intro[V](
    g: SimpleGraph[V], s: FiniteSet[V], t: FiniteSet[V]
) {
    t.subset_eq(s) and is_independent_in(g, t) implies is_independent_subset(g, s, t)
} by {
    if t.subset_eq(s) and is_independent_in(g, t) {
        is_independent_subset(g, s, t) = (t.subset_eq(s) and is_independent_in(g, t))
        is_independent_subset(g, s, t)
    }
}

/// True if `t` is independent and no vertex of `s` can be added to it.
define is_maximal_independent_in[V](
    g: SimpleGraph[V], s: FiniteSet[V], t: FiniteSet[V]
) -> Bool {
    is_independent_subset(g, s, t) and forall(v: V) {
        s.contains(v) and not t.contains(v) implies not is_independent_in(g, t.insert(v))
    }
}

/// A maximal independent set is independent, and admits no new vertex.
theorem is_maximal_independent_in_apply[V](
    g: SimpleGraph[V], s: FiniteSet[V], t: FiniteSet[V], v: V
) {
    is_maximal_independent_in(g, s, t) and s.contains(v) and not t.contains(v)
        implies not is_independent_in(g, t.insert(v))
} by {
    if is_maximal_independent_in(g, s, t) and s.contains(v) and not t.contains(v) {
        is_maximal_independent_in(g, s, t) = (is_independent_subset(g, s, t) and forall(u: V) {
            s.contains(u) and not t.contains(u) implies not is_independent_in(g, t.insert(u))
        })
        forall(u: V) {
            s.contains(u) and not t.contains(u) implies not is_independent_in(g, t.insert(u))
        }
        (s.contains(v) and not t.contains(v) implies not is_independent_in(g, t.insert(v)))
        not is_independent_in(g, t.insert(v))
    }
}

/// A maximal independent set is an independent subset.
theorem is_maximal_independent_in_subset[V](
    g: SimpleGraph[V], s: FiniteSet[V], t: FiniteSet[V]
) {
    is_maximal_independent_in(g, s, t) implies is_independent_subset(g, s, t)
} by {
    if is_maximal_independent_in(g, s, t) {
        is_maximal_independent_in(g, s, t) = (is_independent_subset(g, s, t) and forall(u: V) {
            s.contains(u) and not t.contains(u) implies not is_independent_in(g, t.insert(u))
        })
        is_independent_subset(g, s, t)
    }
}

/// The two conditions give a maximal independent set.
theorem is_maximal_independent_in_intro[V](
    g: SimpleGraph[V], s: FiniteSet[V], t: FiniteSet[V]
) {
    is_independent_subset(g, s, t) and (forall(v: V) {
        s.contains(v) and not t.contains(v) implies not is_independent_in(g, t.insert(v))
    }) implies is_maximal_independent_in(g, s, t)
} by {
    if is_independent_subset(g, s, t) and forall(v: V) {
        s.contains(v) and not t.contains(v) implies not is_independent_in(g, t.insert(v))
    } {
        is_maximal_independent_in(g, s, t) = (is_independent_subset(g, s, t) and forall(u: V) {
            s.contains(u) and not t.contains(u) implies not is_independent_in(g, t.insert(u))
        })
        is_maximal_independent_in(g, s, t)
    }
}

/// Adding an outside vertex to an independent set can only create edges at that vertex.
///
/// So if the enlarged set fails to be independent, the new vertex has a neighbor in the old
/// one. This is the step that turns maximality into domination.
theorem independent_insert_adj_at_new[V](
    g: SimpleGraph[V], t: FiniteSet[V], v: V
) {
    is_independent_in(g, t) and not is_independent_in(g, t.insert(v))
        implies exists(w: V) { t.contains(w) and g.adj(w, v) }
} by {
    if is_independent_in(g, t) and not is_independent_in(g, t.insert(v)) {
        if forall(x: V, y: V) {
            t.insert(v).contains(x) and t.insert(v).contains(y) implies not g.adj(x, y)
        } {
            is_independent_in_intro(g, t.insert(v))
            is_independent_in(g, t.insert(v))
            false
        }
        exists(x: V, y: V) {
            t.insert(v).contains(x) and t.insert(v).contains(y) and g.adj(x, y)
        }
        let (a: V, b: V) satisfy {
            t.insert(v).contains(a) and t.insert(v).contains(b) and g.adj(a, b)
        }
        fs_insert_contains_cases(t, v, a)
        fs_insert_contains_cases(t, v, b)
        if a = v {
            if b = v {
                g.adj(v, v)
                simple_graph_adj_irreflexive(g, v)
                not g.adj(v, v)
                false
            }
            if b != v {
                t.contains(b)
                g.adj(v, b)
                simple_graph_adj_comm(g, v, b)
                g.adj(b, v)
                exists(w: V) { t.contains(w) and g.adj(w, v) }
            }
            exists(w: V) { t.contains(w) and g.adj(w, v) }
        }
        if a != v {
            t.contains(a)
            if b = v {
                g.adj(a, v)
                exists(w: V) { t.contains(w) and g.adj(w, v) }
            }
            if b != v {
                t.contains(b)
                is_independent_in_apply(g, t, a, b)
                not g.adj(a, b)
                false
            }
            exists(w: V) { t.contains(w) and g.adj(w, v) }
        }
        exists(w: V) { t.contains(w) and g.adj(w, v) }
    }
}

/// A maximal independent set is dominating.
///
/// This is the classical bridge between the two invariants: a vertex outside a maximal
/// independent set cannot be added, so it must already have a neighbor inside.
theorem maximal_independent_is_dominating[V](
    g: SimpleGraph[V], s: FiniteSet[V], t: FiniteSet[V]
) {
    is_maximal_independent_in(g, s, t) implies is_dominating_set(g, s, t)
} by {
    if is_maximal_independent_in(g, s, t) {
        is_maximal_independent_in_subset(g, s, t)
        is_independent_subset(g, s, t)
        is_independent_subset_apply(g, s, t)
        is_independent_in(g, t)
        forall(v: V) {
            if s.contains(v) {
                if t.contains(v) {
                    is_dominated_of_contains(g, t, v)
                    is_dominated(g, t, v)
                }
                if not t.contains(v) {
                    is_maximal_independent_in_apply(g, s, t, v)
                    not is_independent_in(g, t.insert(v))
                    independent_insert_adj_at_new(g, t, v)
                    let (w: V) satisfy {
                        t.contains(w) and g.adj(w, v)
                    }
                    is_dominated_of_adj(g, t, v, w)
                    is_dominated(g, t, v)
                }
                is_dominated(g, t, v)
            }
            s.contains(v) implies is_dominated(g, t, v)
        }
        is_dominating_set_intro(g, s, t)
        is_dominating_set(g, s, t)
    }
}

/// The domination number is at most the size of any maximal independent set.
///
/// The independent domination number sits between the two, so this is the lower half of the
/// classical chain relating them.
theorem domination_number_le_maximal_independent[V](
    g: SimpleGraph[V], s: FiniteSet[V], t: FiniteSet[V]
) {
    is_maximal_independent_in(g, s, t) implies domination_number(g, s) <= fs_card(t)
} by {
    if is_maximal_independent_in(g, s, t) {
        is_maximal_independent_in_subset(g, s, t)
        is_independent_subset(g, s, t)
        is_independent_subset_apply(g, s, t)
        t.subset_eq(s)
        maximal_independent_is_dominating(g, s, t)
        is_dominating_set(g, s, t)
        domination_number_is_least(g, s, t)
        domination_number(g, s) <= fs_card(t)
    }
}

/// True if `t` both dominates `s` and is independent.
define is_independent_dominating_set[V](
    g: SimpleGraph[V], s: FiniteSet[V], t: FiniteSet[V]
) -> Bool {
    is_independent_subset(g, s, t) and is_dominating_set(g, s, t)
}

/// An independent dominating set is independent and dominating.
theorem is_independent_dominating_set_apply[V](
    g: SimpleGraph[V], s: FiniteSet[V], t: FiniteSet[V]
) {
    is_independent_dominating_set(g, s, t)
        implies is_independent_subset(g, s, t) and is_dominating_set(g, s, t)
} by {
    if is_independent_dominating_set(g, s, t) {
        is_independent_dominating_set(g, s, t) =
            (is_independent_subset(g, s, t) and is_dominating_set(g, s, t))
        is_independent_subset(g, s, t) and is_dominating_set(g, s, t)
    }
}

/// The two conditions give an independent dominating set.
theorem is_independent_dominating_set_intro[V](
    g: SimpleGraph[V], s: FiniteSet[V], t: FiniteSet[V]
) {
    is_independent_subset(g, s, t) and is_dominating_set(g, s, t)
        implies is_independent_dominating_set(g, s, t)
} by {
    if is_independent_subset(g, s, t) and is_dominating_set(g, s, t) {
        is_independent_dominating_set(g, s, t) =
            (is_independent_subset(g, s, t) and is_dominating_set(g, s, t))
        is_independent_dominating_set(g, s, t)
    }
}

/// A maximal independent set is an independent dominating set.
theorem maximal_independent_is_independent_dominating[V](
    g: SimpleGraph[V], s: FiniteSet[V], t: FiniteSet[V]
) {
    is_maximal_independent_in(g, s, t) implies is_independent_dominating_set(g, s, t)
} by {
    if is_maximal_independent_in(g, s, t) {
        is_maximal_independent_in_subset(g, s, t)
        is_independent_subset(g, s, t)
        maximal_independent_is_dominating(g, s, t)
        is_dominating_set(g, s, t)
        is_independent_dominating_set_intro(g, s, t)
        is_independent_dominating_set(g, s, t)
    }
}

/// The domination number is at most the size of any independent dominating set.
theorem domination_number_le_independent_dominating[V](
    g: SimpleGraph[V], s: FiniteSet[V], t: FiniteSet[V]
) {
    is_independent_dominating_set(g, s, t) implies domination_number(g, s) <= fs_card(t)
} by {
    if is_independent_dominating_set(g, s, t) {
        is_independent_dominating_set_apply(g, s, t)
        is_independent_subset(g, s, t)
        is_dominating_set(g, s, t)
        is_independent_subset_apply(g, s, t)
        t.subset_eq(s)
        domination_number_is_least(g, s, t)
        domination_number(g, s) <= fs_card(t)
    }
}
