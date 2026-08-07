from nat import Nat
from pair import Pair
from finite_set import FiniteSet, fs_union, fs_insert,
    finite_set_disjoint_union_cardinality_is
from data.finite.finite_set_filter import finite_set_filter, finite_set_filter_contains_eq
from data.finite.finite_set_card import fs_card, fs_card_eq_of_cardinality_is, fs_card_cardinality_is
from data.finite.finite_set_membership import fs_insert_contains_eq
from graph.simple_graph import SimpleGraph
from graph.simple_graph_edges import directed_edges, directed_edges_contains_eq
from graph.simple_graph_degree_sum import directed_edges_from, directed_edges_from_contains_eq

numerals Nat

/// The count of a disjoint union splits.
theorem fs_card_disjoint_union[T](a: FiniteSet[T], b: FiniteSet[T]) {
    a.is_disjoint(b) implies fs_card(fs_union(a, b)) = fs_card(a) + fs_card(b)
} by {
    if a.is_disjoint(b) {
        fs_card_cardinality_is(a)
        a.cardinality_is(fs_card(a))
        fs_card_cardinality_is(b)
        b.cardinality_is(fs_card(b))
        finite_set_disjoint_union_cardinality_is(a, b, fs_card(a), fs_card(b))
        fs_union(a, b).cardinality_is(fs_card(a) + fs_card(b))
        fs_card_eq_of_cardinality_is(fs_union(a, b), fs_card(a) + fs_card(b))
    }
}

/// True of the pairs whose first component lies in `t`.
define first_in[V](t: FiniteSet[V]) -> (Pair[V, V] -> Bool) {
    function(p: Pair[V, V]) {
        t.contains(p.first)
    }
}

/// A pair has its first component in `t` exactly when the predicate holds.
theorem first_in_eq[V](t: FiniteSet[V], p: Pair[V, V]) {
    first_in(t)(p) = t.contains(p.first)
}

/// The directed edges leaving any vertex of `t`.
///
/// The ambient set `s` stays fixed while `t` varies, which is what makes induction on `t`
/// legitimate. Inducting on the vertex set itself would change every degree.
define directed_edges_from_set[V](
    g: SimpleGraph[V], s: FiniteSet[V], t: FiniteSet[V]
) -> FiniteSet[Pair[V, V]] {
    finite_set_filter(directed_edges(g, s), first_in(t))
}

/// A pair leaves `t` exactly when it is a directed edge starting in `t`.
theorem directed_edges_from_set_contains_eq[V](
    g: SimpleGraph[V], s: FiniteSet[V], t: FiniteSet[V], p: Pair[V, V]
) {
    directed_edges_from_set(g, s, t).contains(p) = (directed_edges(g, s).contains(p) and t.contains(p.first))
} by {
    finite_set_filter_contains_eq(directed_edges(g, s), first_in(t), p)
    directed_edges_from_set(g, s, t).contains(p) = (directed_edges(g, s).contains(p) and first_in(t)(p))
    first_in_eq(t, p)
    first_in(t)(p) = t.contains(p.first)
}

/// An edge leaving an enlarged set leaves the new vertex or the old set.
theorem directed_edges_from_set_insert_forward[V](
    g: SimpleGraph[V], s: FiniteSet[V], t: FiniteSet[V], v: V, p: Pair[V, V]
) {
    directed_edges_from_set(g, s, fs_insert(t, v)).contains(p) implies directed_edges_from(g, s, v).contains(p) or directed_edges_from_set(g, s, t).contains(p)
} by {
    if directed_edges_from_set(g, s, fs_insert(t, v)).contains(p) {
        directed_edges_from_set_contains_eq(g, s, fs_insert(t, v), p)
        directed_edges(g, s).contains(p)
        fs_insert(t, v).contains(p.first)
        fs_insert_contains_eq(t, v, p.first)
        p.first = v or t.contains(p.first)
        if p.first = v {
            directed_edges_from_contains_eq(g, s, v, p)
            directed_edges_from(g, s, v).contains(p)
        }
        if p.first != v {
            t.contains(p.first)
            directed_edges_from_set_contains_eq(g, s, t, p)
            directed_edges_from_set(g, s, t).contains(p)
        }
        directed_edges_from(g, s, v).contains(p) or directed_edges_from_set(g, s, t).contains(p)
    }
}

/// An edge leaving the new vertex or the old set leaves the enlarged set.
theorem directed_edges_from_set_insert_backward[V](
    g: SimpleGraph[V], s: FiniteSet[V], t: FiniteSet[V], v: V, p: Pair[V, V]
) {
    directed_edges_from(g, s, v).contains(p) or directed_edges_from_set(g, s, t).contains(p) implies directed_edges_from_set(g, s, fs_insert(t, v)).contains(p)
} by {
    if directed_edges_from(g, s, v).contains(p) or directed_edges_from_set(g, s, t).contains(p) {
        if directed_edges_from(g, s, v).contains(p) {
            directed_edges_from_contains_eq(g, s, v, p)
            directed_edges(g, s).contains(p)
            p.first = v
            fs_insert_contains_eq(t, v, p.first)
            fs_insert(t, v).contains(p.first)
            directed_edges_from_set_contains_eq(g, s, fs_insert(t, v), p)
            directed_edges_from_set(g, s, fs_insert(t, v)).contains(p)
        }
        if not directed_edges_from(g, s, v).contains(p) {
            directed_edges_from_set(g, s, t).contains(p)
            directed_edges_from_set_contains_eq(g, s, t, p)
            directed_edges(g, s).contains(p)
            t.contains(p.first)
            fs_insert_contains_eq(t, v, p.first)
            fs_insert(t, v).contains(p.first)
            directed_edges_from_set_contains_eq(g, s, fs_insert(t, v), p)
            directed_edges_from_set(g, s, fs_insert(t, v)).contains(p)
        }
        directed_edges_from_set(g, s, fs_insert(t, v)).contains(p)
    }
}

/// Edges leaving a fresh vertex are disjoint from those leaving the old set.
///
/// A pair cannot start at `v` and at a vertex of `t` at once when `v` is not in `t`.
theorem directed_edges_from_disjoint[V](
    g: SimpleGraph[V], s: FiniteSet[V], t: FiniteSet[V], v: V
) {
    not t.contains(v) implies directed_edges_from(g, s, v).is_disjoint(directed_edges_from_set(g, s, t))
} by {
    if not t.contains(v) {
        forall(p: Pair[V, V]) {
            if directed_edges_from(g, s, v).contains(p) and directed_edges_from_set(g, s, t).contains(p) {
                directed_edges_from_contains_eq(g, s, v, p)
                p.first = v
                directed_edges_from_set_contains_eq(g, s, t, p)
                t.contains(p.first)
                t.contains(v)
                false
            }
            not (directed_edges_from(g, s, v).contains(p) and directed_edges_from_set(g, s, t).contains(p))
            directed_edges_from(g, s, v).contains(p) = directed_edges_from(g, s, v).underlying_set.contains(p)
            directed_edges_from_set(g, s, t).contains(p) = directed_edges_from_set(g, s, t).underlying_set.contains(p)
            not (directed_edges_from(g, s, v).underlying_set.contains(p) and directed_edges_from_set(g, s, t).underlying_set.contains(p))
        }
        directed_edges_from(g, s, v).underlying_set.is_disjoint(directed_edges_from_set(g, s, t).underlying_set) = forall(p: Pair[V, V]) {
            not (directed_edges_from(g, s, v).underlying_set.contains(p) and directed_edges_from_set(g, s, t).underlying_set.contains(p))
        }
        directed_edges_from(g, s, v).underlying_set.is_disjoint(directed_edges_from_set(g, s, t).underlying_set)
        directed_edges_from(g, s, v).is_disjoint(directed_edges_from_set(g, s, t))
    }
}
