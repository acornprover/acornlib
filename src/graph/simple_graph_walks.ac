from nat import Nat, alt_induction
from data.basic.functions import injective_fn_eq
from list import List, injective_map_is_unique, map, map_length
from graph.simple_graph import SimpleGraph, SimpleGraphHom, SimpleGraphEmbedding,
    simple_graph_embedding_is_injective, simple_graph_embedding_map_adj_iff,
    simple_graph_embedding_maps_adj, simple_graph_hom_maps_adj
from data.basic.relation_basic import relation_compose_has_witness
from data.basic.relation import relation_power, relation_power_zero, relation_power_suc
from graph.simple_graph_connectivity import simple_graph_connected, simple_graph_reachable,
    simple_graph_reachable_of_adj, simple_graph_reachable_refl,
    simple_graph_reachable_transitive

numerals Nat

/// A walk from `start` to `finish` whose edge targets are stored in `steps`.
/// The empty target list is the length-zero walk, so it requires `start = finish`.
define simple_graph_walk[V](g: SimpleGraph[V], start: V, finish: V, steps: List[V]) -> Bool {
    match steps {
        List.nil[V] {
            start = finish
        }
        List.cons(next, rest) {
            g.adj(start, next) and simple_graph_walk(g, next, finish, rest)
        }
    }
}

/// The full vertex list underlying a walk: the initial vertex followed by all edge targets.
define simple_graph_walk_vertices[V](start: V, steps: List[V]) -> List[V] {
    List.cons(start, steps)
}

/// A path is a walk whose full vertex list has no repeated vertices.
define simple_graph_path[V](g: SimpleGraph[V], start: V, finish: V, steps: List[V]) -> Bool {
    simple_graph_walk(g, start, finish, steps) and simple_graph_walk_vertices(start, steps).is_unique
}

/// A closed walk is a positive-length walk from a vertex back to itself.
define simple_graph_closed_walk[V](g: SimpleGraph[V], start: V, steps: List[V]) -> Bool {
    simple_graph_walk(g, start, start, steps) and steps.length > Nat.0
}

/// The length-zero walk stays at its start vertex.
theorem simple_graph_walk_nil[V](g: SimpleGraph[V], start: V, finish: V) {
    simple_graph_walk(g, start, finish, List.nil[V]) = (start = finish)
} by {
    if simple_graph_walk(g, start, finish, List.nil[V]) {
        start = finish
    }
    if start = finish {
        simple_graph_walk(g, start, finish, List.nil[V])
    }
}

/// Every vertex has a length-zero walk to itself.
theorem simple_graph_walk_refl[V](g: SimpleGraph[V], x: V) {
    simple_graph_walk(g, x, x, List.nil[V])
} by {
    simple_graph_walk_nil(g, x, x)
}

/// A nonempty walk is exactly a first edge followed by a tail walk.
theorem simple_graph_walk_cons_iff[V](g: SimpleGraph[V], start: V, finish: V, next: V, rest: List[V]) {
    simple_graph_walk(g, start, finish, List.cons(next, rest)) =
        (g.adj(start, next) and simple_graph_walk(g, next, finish, rest))
} by {
    if simple_graph_walk(g, start, finish, List.cons(next, rest)) {
        g.adj(start, next)
        simple_graph_walk(g, next, finish, rest)
        g.adj(start, next) and simple_graph_walk(g, next, finish, rest)
    }
    if g.adj(start, next) and simple_graph_walk(g, next, finish, rest) {
        simple_graph_walk(g, start, finish, List.cons(next, rest))
    }
}

/// A single graph edge is a one-step walk.
theorem simple_graph_walk_of_adj[V](g: SimpleGraph[V], start: V, finish: V) {
    g.adj(start, finish) implies simple_graph_walk(g, start, finish, List.singleton(finish))
} by {
    if g.adj(start, finish) {
        List.singleton(finish) = List.cons(finish, List.nil[V])
        simple_graph_walk(g, finish, finish, List.nil[V])
        simple_graph_walk(g, start, finish, List.cons(finish, List.nil[V]))
        simple_graph_walk(g, start, finish, List.singleton(finish))
    }
}

/// A graph homomorphism maps walks to walks.
theorem simple_graph_hom_maps_walk[V, W](f: SimpleGraphHom[V, W], start: V, finish: V, steps: List[V]) {
    simple_graph_walk(f.src, start, finish, steps) implies
        simple_graph_walk(f.dst, f.map(start), f.map(finish), map(steps, f.map))
} by {
    define pc(xs: List[V], s: V, t: V) -> Bool {
        simple_graph_walk(f.src, s, t, xs) implies
            simple_graph_walk(f.dst, f.map(s), f.map(t), map(xs, f.map))
    }

    define p(xs: List[V]) -> Bool {
        forall(s: V, t: V) {
            pc(xs, s, t)
        }
    }

    forall(s: V, t: V) {
        if simple_graph_walk(f.src, s, t, List.nil[V]) {
            simple_graph_walk_nil(f.src, s, t)
            s = t
            map[V, W](List.nil[V], f.map) = List.nil[W]
            simple_graph_walk(f.dst, f.map(s), f.map(t), List.nil[W])
        }
        pc(List.nil[V], s, t)
    }
    p(List.nil[V])

    forall(next: V, tail: List[V]) {
        if p(tail) {
            forall(s: V, t: V) {
                if simple_graph_walk(f.src, s, t, List.cons(next, tail)) {
                    simple_graph_walk_cons_iff(f.src, s, t, next, tail)
                    f.src.adj(s, next)
                    simple_graph_walk(f.src, next, t, tail)
                    simple_graph_hom_maps_adj(f, s, next)
                    f.dst.adj(f.map(s), f.map(next))
                    pc(tail, next, t)
                    pc(tail, next, t) = (
                        simple_graph_walk(f.src, next, t, tail) implies
                            simple_graph_walk(f.dst, f.map(next), f.map(t), map(tail, f.map))
                    )
                    simple_graph_walk(f.dst, f.map(next), f.map(t), map(tail, f.map))
                    map[V, W](List.cons(next, tail), f.map) = List.cons(f.map(next), map(tail, f.map))
                    simple_graph_walk(f.dst, f.map(s), f.map(t), List.cons(f.map(next), map(tail, f.map)))
                    simple_graph_walk(f.dst, f.map(s), f.map(t), map(List.cons(next, tail), f.map))
                }
                pc(List.cons(next, tail), s, t)
            }
            p(List.cons(next, tail))
        }
    }
    forall(next: V, tail: List[V]) { p(tail) implies p(List.cons(next, tail)) }
    function[T0](x0: List[T0] -> Bool) {
        not forall(x1: T0, x2: List[T0]) { not x0(x2) or x0(List.cons(x1, x2)) } or
        not x0(List.nil[T0]) or forall(x3: List[T0]) { x0(x3) } = true
    }[V](p)
    function[T0](x0: List[T0] -> Bool, x1: List[T0]) {
        not forall(x2: T0, x3: List[T0]) { not x0(x3) or x0(List.cons(x2, x3)) } or
        not x0(List.nil[T0]) or x0(x1)
    }[V](p, steps)
    p(steps)
    pc(steps, start, finish)
    if simple_graph_walk(f.src, start, finish, steps) {
        pc(steps, start, finish) = (
            simple_graph_walk(f.src, start, finish, steps) implies
                simple_graph_walk(f.dst, f.map(start), f.map(finish), map(steps, f.map))
        )
        simple_graph_walk(f.dst, f.map(start), f.map(finish), map(steps, f.map))
    }
}

/// A graph embedding maps walks to walks.
theorem simple_graph_embedding_maps_walk[V, W](f: SimpleGraphEmbedding[V, W], start: V, finish: V, steps: List[V]) {
    simple_graph_walk(f.src, start, finish, steps) implies
        simple_graph_walk(f.dst, f.map(start), f.map(finish), map(steps, f.map))
} by {
    define pc(xs: List[V], s: V, t: V) -> Bool {
        simple_graph_walk(f.src, s, t, xs) implies
            simple_graph_walk(f.dst, f.map(s), f.map(t), map(xs, f.map))
    }

    define p(xs: List[V]) -> Bool {
        forall(s: V, t: V) {
            pc(xs, s, t)
        }
    }

    forall(s: V, t: V) {
        if simple_graph_walk(f.src, s, t, List.nil[V]) {
            simple_graph_walk_nil(f.src, s, t)
            s = t
            map[V, W](List.nil[V], f.map) = List.nil[W]
            simple_graph_walk(f.dst, f.map(s), f.map(t), List.nil[W])
        }
        pc(List.nil[V], s, t)
    }
    p(List.nil[V])

    forall(next: V, tail: List[V]) {
        if p(tail) {
            forall(s: V, t: V) {
                if simple_graph_walk(f.src, s, t, List.cons(next, tail)) {
                    simple_graph_walk_cons_iff(f.src, s, t, next, tail)
                    f.src.adj(s, next)
                    simple_graph_walk(f.src, next, t, tail)
                    simple_graph_embedding_maps_adj(f, s, next)
                    f.dst.adj(f.map(s), f.map(next))
                    pc(tail, next, t)
                    pc(tail, next, t) = (
                        simple_graph_walk(f.src, next, t, tail) implies
                            simple_graph_walk(f.dst, f.map(next), f.map(t), map(tail, f.map))
                    )
                    simple_graph_walk(f.dst, f.map(next), f.map(t), map(tail, f.map))
                    map[V, W](List.cons(next, tail), f.map) = List.cons(f.map(next), map(tail, f.map))
                    simple_graph_walk(f.dst, f.map(s), f.map(t), List.cons(f.map(next), map(tail, f.map)))
                    simple_graph_walk(f.dst, f.map(s), f.map(t), map(List.cons(next, tail), f.map))
                }
                pc(List.cons(next, tail), s, t)
            }
            p(List.cons(next, tail))
        }
    }
    forall(next: V, tail: List[V]) { p(tail) implies p(List.cons(next, tail)) }
    function[T0](x0: List[T0] -> Bool) {
        not forall(x1: T0, x2: List[T0]) { not x0(x2) or x0(List.cons(x1, x2)) } or
        not x0(List.nil[T0]) or forall(x3: List[T0]) { x0(x3) } = true
    }[V](p)
    function[T0](x0: List[T0] -> Bool, x1: List[T0]) {
        not forall(x2: T0, x3: List[T0]) { not x0(x3) or x0(List.cons(x2, x3)) } or
        not x0(List.nil[T0]) or x0(x1)
    }[V](p, steps)
    p(steps)
    pc(steps, start, finish)
    if simple_graph_walk(f.src, start, finish, steps) {
        pc(steps, start, finish) = (
            simple_graph_walk(f.src, start, finish, steps) implies
                simple_graph_walk(f.dst, f.map(start), f.map(finish), map(steps, f.map))
        )
        simple_graph_walk(f.dst, f.map(start), f.map(finish), map(steps, f.map))
    }
}

/// A graph embedding reflects walks whose target list is pointwise mapped from the source.
theorem simple_graph_embedding_reflects_mapped_walk[V, W](
    f: SimpleGraphEmbedding[V, W], start: V, finish: V, steps: List[V]
) {
    simple_graph_walk(f.dst, f.map(start), f.map(finish), map(steps, f.map)) implies
        simple_graph_walk(f.src, start, finish, steps)
} by {
    define pc(xs: List[V], s: V, t: V) -> Bool {
        simple_graph_walk(f.dst, f.map(s), f.map(t), map(xs, f.map)) implies
            simple_graph_walk(f.src, s, t, xs)
    }

    define p(xs: List[V]) -> Bool {
        forall(s: V, t: V) {
            pc(xs, s, t)
        }
    }

    forall(s: V, t: V) {
        if simple_graph_walk(f.dst, f.map(s), f.map(t), map(List.nil[V], f.map)) {
            map[V, W](List.nil[V], f.map) = List.nil[W]
            simple_graph_walk_nil(f.dst, f.map(s), f.map(t))
            f.map(s) = f.map(t)
            simple_graph_embedding_is_injective(f)
            injective_fn_eq(f.map, s, t)
            s = t
            simple_graph_walk(f.src, s, t, List.nil[V])
        }
        pc(List.nil[V], s, t)
    }
    p(List.nil[V])

    forall(next: V, tail: List[V]) {
        if p(tail) {
            forall(s: V, t: V) {
                if simple_graph_walk(f.dst, f.map(s), f.map(t), map(List.cons(next, tail), f.map)) {
                    map[V, W](List.cons(next, tail), f.map) = List.cons(f.map(next), map(tail, f.map))
                    simple_graph_walk(f.dst, f.map(s), f.map(t), List.cons(f.map(next), map(tail, f.map)))
                    simple_graph_walk_cons_iff(f.dst, f.map(s), f.map(t), f.map(next), map(tail, f.map))
                    f.dst.adj(f.map(s), f.map(next))
                    simple_graph_walk(f.dst, f.map(next), f.map(t), map(tail, f.map))
                    simple_graph_embedding_map_adj_iff(f, s, next)
                    f.src.adj(s, next)
                    pc(tail, next, t)
                    pc(tail, next, t) = (
                        simple_graph_walk(f.dst, f.map(next), f.map(t), map(tail, f.map)) implies
                            simple_graph_walk(f.src, next, t, tail)
                    )
                    simple_graph_walk(f.src, next, t, tail)
                    simple_graph_walk(f.src, s, t, List.cons(next, tail))
                }
                pc(List.cons(next, tail), s, t)
            }
            p(List.cons(next, tail))
        }
    }
    forall(next: V, tail: List[V]) { p(tail) implies p(List.cons(next, tail)) }
    function[T0](x0: List[T0] -> Bool) {
        not forall(x1: T0, x2: List[T0]) { not x0(x2) or x0(List.cons(x1, x2)) } or
        not x0(List.nil[T0]) or forall(x3: List[T0]) { x0(x3) } = true
    }[V](p)
    function[T0](x0: List[T0] -> Bool, x1: List[T0]) {
        not forall(x2: T0, x3: List[T0]) { not x0(x3) or x0(List.cons(x2, x3)) } or
        not x0(List.nil[T0]) or x0(x1)
    }[V](p, steps)
    p(steps)
    pc(steps, start, finish)
    if simple_graph_walk(f.dst, f.map(start), f.map(finish), map(steps, f.map)) {
        pc(steps, start, finish) = (
            simple_graph_walk(f.dst, f.map(start), f.map(finish), map(steps, f.map)) implies
                simple_graph_walk(f.src, start, finish, steps)
        )
        simple_graph_walk(f.src, start, finish, steps)
    }
}

/// Every explicit walk gives reachability in the accepted connectivity relation.
theorem simple_graph_walk_imp_reachable[V](g: SimpleGraph[V], start: V, finish: V, steps: List[V]) {
    simple_graph_walk(g, start, finish, steps) implies simple_graph_reachable(g, start, finish)
} by {
    define pc(xs: List[V], s: V, t: V) -> Bool {
        simple_graph_walk(g, s, t, xs) implies simple_graph_reachable(g, s, t)
    }

    define p(xs: List[V]) -> Bool {
        forall(s: V, t: V) {
            pc(xs, s, t)
        }
    }

    forall(s: V, t: V) {
        if simple_graph_walk(g, s, t, List.nil[V]) {
            simple_graph_walk_nil(g, s, t)
            s = t
            simple_graph_reachable_refl(g, s)
            simple_graph_reachable(g, s, t)
        }
        pc(List.nil[V], s, t)
    }
    p(List.nil[V])

    forall(next: V, tail: List[V]) {
        if p(tail) {
            forall(s: V, t: V) {
                if simple_graph_walk(g, s, t, List.cons(next, tail)) {
                    simple_graph_walk_cons_iff(g, s, t, next, tail)
                    g.adj(s, next)
                    simple_graph_walk(g, next, t, tail)
                    simple_graph_reachable_of_adj(g, s, next)
                    simple_graph_reachable(g, s, next)
                    pc(tail, next, t)
                    pc(tail, next, t) = (simple_graph_walk(g, next, t, tail) implies simple_graph_reachable(g, next, t))
                    simple_graph_reachable(g, next, t)
                    simple_graph_reachable_transitive(g, s, next, t)
                    simple_graph_reachable(g, s, t)
                }
                pc(List.cons(next, tail), s, t)
            }
            p(List.cons(next, tail))
        }
    }
    forall(next: V, tail: List[V]) { p(tail) implies p(List.cons(next, tail)) }
    function[T0](x0: List[T0] -> Bool) {
        not forall(x1: T0, x2: List[T0]) { not x0(x2) or x0(List.cons(x1, x2)) } or
        not x0(List.nil[T0]) or forall(x3: List[T0]) { x0(x3) } = true
    }[V](p)
    function[T0](x0: List[T0] -> Bool, x1: List[T0]) {
        not forall(x2: T0, x3: List[T0]) { not x0(x3) or x0(List.cons(x2, x3)) } or
        not x0(List.nil[T0]) or x0(x1)
    }[V](p, steps)
    p(steps)
    pc(steps, start, finish)
    if simple_graph_walk(g, start, finish, steps) {
        pc(steps, start, finish) = (simple_graph_walk(g, start, finish, steps) implies simple_graph_reachable(g, start, finish))
        simple_graph_reachable(g, start, finish)
    }
}

/// Appending one adjacent target to the target list extends a walk by one edge.
theorem simple_graph_walk_append_adj[V](g: SimpleGraph[V], start: V, mid: V, finish: V, steps: List[V]) {
    simple_graph_walk(g, start, mid, steps) and g.adj(mid, finish) implies
    simple_graph_walk(g, start, finish, steps.append(finish))
} by {
    define pc(xs: List[V], s: V, m: V, t: V) -> Bool {
        simple_graph_walk(g, s, m, xs) and g.adj(m, t) implies
        simple_graph_walk(g, s, t, xs.append(t))
    }

    define p(xs: List[V]) -> Bool {
        forall(s: V, m: V, t: V) {
            pc(xs, s, m, t)
        }
    }

    forall(s: V, m: V, t: V) {
        if simple_graph_walk(g, s, m, List.nil[V]) and g.adj(m, t) {
            simple_graph_walk_nil(g, s, m)
            s = m
            g.adj(s, t)
            simple_graph_walk_of_adj(g, s, t)
            simple_graph_walk(g, s, t, List.singleton(t))
            List.nil[V].append(t) = List.nil[V] + List.singleton(t)
            List.nil[V] + List.singleton(t) = List.singleton(t)
            simple_graph_walk(g, s, t, List.nil[V].append(t))
        }
        pc(List.nil[V], s, m, t)
    }
    p(List.nil[V])

    forall(next: V, tail: List[V]) {
        if p(tail) {
            forall(s: V, m: V, t: V) {
                if simple_graph_walk(g, s, m, List.cons(next, tail)) and g.adj(m, t) {
                    simple_graph_walk_cons_iff(g, s, m, next, tail)
                    g.adj(s, next)
                    simple_graph_walk(g, next, m, tail)
                    pc(tail, next, m, t)
                    pc(tail, next, m, t) = (
                        simple_graph_walk(g, next, m, tail) and g.adj(m, t) implies
                        simple_graph_walk(g, next, t, tail.append(t))
                    )
                    simple_graph_walk(g, next, t, tail.append(t))
                    List.cons(next, tail).append(t) = List.cons(next, tail) + List.singleton(t)
                    List.cons(next, tail) + List.singleton(t) = List.cons(next, tail + List.singleton(t))
                    tail.append(t) = tail + List.singleton(t)
                    List.cons(next, tail).append(t) = List.cons(next, tail.append(t))
                    simple_graph_walk(g, s, t, List.cons(next, tail.append(t)))
                    simple_graph_walk(g, s, t, List.cons(next, tail).append(t))
                }
                pc(List.cons(next, tail), s, m, t)
            }
            p(List.cons(next, tail))
        }
    }
    forall(next: V, tail: List[V]) { p(tail) implies p(List.cons(next, tail)) }
    function[T0](x0: List[T0] -> Bool) {
        not forall(x1: T0, x2: List[T0]) { not x0(x2) or x0(List.cons(x1, x2)) } or
        not x0(List.nil[T0]) or forall(x3: List[T0]) { x0(x3) } = true
    }[V](p)
    function[T0](x0: List[T0] -> Bool, x1: List[T0]) {
        not forall(x2: T0, x3: List[T0]) { not x0(x3) or x0(List.cons(x2, x3)) } or
        not x0(List.nil[T0]) or x0(x1)
    }[V](p, steps)
    p(steps)
    pc(steps, start, mid, finish)
    if simple_graph_walk(g, start, mid, steps) and g.adj(mid, finish) {
        pc(steps, start, mid, finish) = (
            simple_graph_walk(g, start, mid, steps) and g.adj(mid, finish) implies
            simple_graph_walk(g, start, finish, steps.append(finish))
        )
        simple_graph_walk(g, start, finish, steps.append(finish))
    }
}

/// A finite adjacency-relation power is represented by an explicit walk.
theorem simple_graph_relation_power_has_walk[V](g: SimpleGraph[V], n: Nat, start: V, finish: V) {
    relation_power(g.adj, n, start, finish) implies exists(steps: List[V]) {
        simple_graph_walk(g, start, finish, steps)
    }
} by {
    define pc(k: Nat, s: V, t: V) -> Bool {
        relation_power(g.adj, k, s, t) implies exists(steps: List[V]) {
            simple_graph_walk(g, s, t, steps)
        }
    }

    define p(k: Nat) -> Bool {
        forall(s: V, t: V) {
            pc(k, s, t)
        }
    }

    forall(s: V, t: V) {
        if relation_power(g.adj, Nat.zero, s, t) {
            relation_power_zero(g.adj)
            s = t
            simple_graph_walk_refl(g, s)
            exists(steps: List[V]) {
                steps = List.nil[V] and simple_graph_walk(g, s, t, steps)
            }
        }
        pc(Nat.zero, s, t)
    }
    p(Nat.zero)

    forall(k: Nat) {
        if p(k) {
            forall(s: V, t: V) {
                if relation_power(g.adj, k.suc, s, t) {
                    relation_power_suc(g.adj, k)
                    relation_compose_has_witness(relation_power(g.adj, k), g.adj, s, t)
                    let m: V satisfy {
                        relation_power(g.adj, k, s, m) and g.adj(m, t)
                    }
                    pc(k, s, m)
                    pc(k, s, m) = (
                        relation_power(g.adj, k, s, m) implies exists(steps: List[V]) {
                            simple_graph_walk(g, s, m, steps)
                        }
                    )
                    let initial_steps: List[V] satisfy {
                        simple_graph_walk(g, s, m, initial_steps)
                    }
                    simple_graph_walk_append_adj(g, s, m, t, initial_steps)
                    simple_graph_walk(g, s, t, initial_steps.append(t))
                    exists(steps: List[V]) {
                        steps = initial_steps.append(t) and simple_graph_walk(g, s, t, steps)
                    }
                }
                pc(k.suc, s, t)
            }
            p(k.suc)
        }
    }
    forall(k: Nat) { p(k) implies p(k.suc) }
    p(Nat.zero) and forall(k: Nat) { p(k) implies p(k.suc) }
    alt_induction(p)
    forall(k: Nat) { p(k) }
    p(n)
    pc(n, start, finish)
    if relation_power(g.adj, n, start, finish) {
        pc(n, start, finish) = (relation_power(g.adj, n, start, finish) implies exists(steps: List[V]) {
            simple_graph_walk(g, start, finish, steps)
        })
        exists(steps: List[V]) {
            simple_graph_walk(g, start, finish, steps)
        }
    }
}

/// Reachability has an explicit walk witness.
theorem simple_graph_reachable_has_walk[V](g: SimpleGraph[V], start: V, finish: V) {
    simple_graph_reachable(g, start, finish) implies exists(steps: List[V]) {
        simple_graph_walk(g, start, finish, steps)
    }
} by {
    if simple_graph_reachable(g, start, finish) {
        let n: Nat satisfy {
            relation_power(g.adj, n, start, finish)
        }
        simple_graph_relation_power_has_walk(g, n, start, finish)
        exists(steps: List[V]) {
            simple_graph_walk(g, start, finish, steps)
        }
    }
}

/// The reachability relation is exactly existence of an explicit walk.
theorem simple_graph_reachable_iff_exists_walk[V](g: SimpleGraph[V], start: V, finish: V) {
    simple_graph_reachable(g, start, finish) = exists(steps: List[V]) {
        simple_graph_walk(g, start, finish, steps)
    }
} by {
    if simple_graph_reachable(g, start, finish) {
        simple_graph_reachable_has_walk(g, start, finish)
        exists(steps: List[V]) {
            simple_graph_walk(g, start, finish, steps)
        }
    }
    if exists(steps: List[V]) { simple_graph_walk(g, start, finish, steps) } {
        let steps: List[V] satisfy {
            simple_graph_walk(g, start, finish, steps)
        }
        simple_graph_walk_imp_reachable(g, start, finish, steps)
        simple_graph_reachable(g, start, finish)
    }
}

/// Connectedness is exactly existence of an explicit walk between every ordered pair of vertices.
theorem simple_graph_connected_iff_forall_walk[V](g: SimpleGraph[V]) {
    simple_graph_connected(g) = forall(start: V, finish: V) {
        exists(steps: List[V]) {
            simple_graph_walk(g, start, finish, steps)
        }
    }
} by {
    if simple_graph_connected(g) {
        forall(start: V, finish: V) {
            simple_graph_connected(g) = forall(x: V, y: V) {
                simple_graph_reachable(g, x, y)
            }
            simple_graph_reachable(g, start, finish)
            simple_graph_reachable_has_walk(g, start, finish)
            exists(steps: List[V]) {
                simple_graph_walk(g, start, finish, steps)
            }
        }
    }
    if forall(start: V, finish: V) { exists(steps: List[V]) { simple_graph_walk(g, start, finish, steps) } } {
        forall(start: V, finish: V) {
            let steps: List[V] satisfy {
                simple_graph_walk(g, start, finish, steps)
            }
            simple_graph_walk_imp_reachable(g, start, finish, steps)
            simple_graph_reachable(g, start, finish)
        }
        simple_graph_connected(g)
    }
}

/// The full vertex list of a path is unique.
theorem simple_graph_path_vertices_unique[V](g: SimpleGraph[V], start: V, finish: V, steps: List[V]) {
    simple_graph_path(g, start, finish, steps) implies simple_graph_walk_vertices(start, steps).is_unique
} by {
    if simple_graph_path(g, start, finish, steps) {
        simple_graph_path(g, start, finish, steps) =
            (simple_graph_walk(g, start, finish, steps) and simple_graph_walk_vertices(start, steps).is_unique)
    }
}

/// A path contains a walk.
theorem simple_graph_path_walk[V](g: SimpleGraph[V], start: V, finish: V, steps: List[V]) {
    simple_graph_path(g, start, finish, steps) implies simple_graph_walk(g, start, finish, steps)
} by {
    if simple_graph_path(g, start, finish, steps) {
        simple_graph_path(g, start, finish, steps) =
            (simple_graph_walk(g, start, finish, steps) and simple_graph_walk_vertices(start, steps).is_unique)
    }
}

/// Every explicit path gives reachability.
theorem simple_graph_path_imp_reachable[V](g: SimpleGraph[V], start: V, finish: V, steps: List[V]) {
    simple_graph_path(g, start, finish, steps) implies simple_graph_reachable(g, start, finish)
} by {
    if simple_graph_path(g, start, finish, steps) {
        simple_graph_path_walk(g, start, finish, steps)
        simple_graph_walk(g, start, finish, steps)
        simple_graph_walk_imp_reachable(g, start, finish, steps)
        simple_graph_reachable(g, start, finish)
    }
}

/// A walk with no repeated vertices is a path.
theorem simple_graph_path_intro[V](g: SimpleGraph[V], start: V, finish: V, steps: List[V]) {
    simple_graph_walk(g, start, finish, steps) and simple_graph_walk_vertices(start, steps).is_unique implies
        simple_graph_path(g, start, finish, steps)
}

/// A graph embedding maps paths to paths.
theorem simple_graph_embedding_maps_path[V, W](f: SimpleGraphEmbedding[V, W], start: V, finish: V, steps: List[V]) {
    simple_graph_path(f.src, start, finish, steps) implies
        simple_graph_path(f.dst, f.map(start), f.map(finish), map(steps, f.map))
} by {
    if simple_graph_path(f.src, start, finish, steps) {
        simple_graph_path_walk(f.src, start, finish, steps)
        simple_graph_path_vertices_unique(f.src, start, finish, steps)
        simple_graph_embedding_maps_walk(f, start, finish, steps)
        simple_graph_embedding_is_injective(f)
        injective_map_is_unique(simple_graph_walk_vertices(start, steps), f.map)
        map(simple_graph_walk_vertices(start, steps), f.map).is_unique
        simple_graph_walk_vertices(start, steps) = List.cons(start, steps)
        map[V, W](List.cons(start, steps), f.map) = List.cons(f.map(start), map(steps, f.map))
        simple_graph_walk_vertices(f.map(start), map(steps, f.map)) = List.cons(f.map(start), map(steps, f.map))
        simple_graph_walk_vertices(f.map(start), map(steps, f.map)).is_unique
        simple_graph_path(f.dst, f.map(start), f.map(finish), map(steps, f.map))
    }
}

/// A closed walk is a walk from its start back to itself.
theorem simple_graph_closed_walk_is_walk[V](g: SimpleGraph[V], start: V, steps: List[V]) {
    simple_graph_closed_walk(g, start, steps) implies simple_graph_walk(g, start, start, steps)
} by {
    if simple_graph_closed_walk(g, start, steps) {
        simple_graph_closed_walk(g, start, steps) =
            (simple_graph_walk(g, start, start, steps) and steps.length > Nat.0)
    }
}

/// A closed walk has positive length.
theorem simple_graph_closed_walk_positive_length[V](g: SimpleGraph[V], start: V, steps: List[V]) {
    simple_graph_closed_walk(g, start, steps) implies steps.length > Nat.0
} by {
    if simple_graph_closed_walk(g, start, steps) {
        simple_graph_closed_walk(g, start, steps) =
            (simple_graph_walk(g, start, start, steps) and steps.length > Nat.0)
    }
}

/// A positive-length walk from a vertex back to itself is a closed walk.
theorem simple_graph_closed_walk_intro[V](g: SimpleGraph[V], start: V, steps: List[V]) {
    simple_graph_walk(g, start, start, steps) and steps.length > Nat.0 implies simple_graph_closed_walk(g, start, steps)
}

/// A graph homomorphism maps closed walks to closed walks.
theorem simple_graph_hom_maps_closed_walk[V, W](f: SimpleGraphHom[V, W], start: V, steps: List[V]) {
    simple_graph_closed_walk(f.src, start, steps) implies simple_graph_closed_walk(f.dst, f.map(start), map(steps, f.map))
} by {
    if simple_graph_closed_walk(f.src, start, steps) {
        simple_graph_closed_walk_is_walk(f.src, start, steps)
        simple_graph_closed_walk_positive_length(f.src, start, steps)
        simple_graph_hom_maps_walk(f, start, start, steps)
        map_length(steps, f.map)
        map(steps, f.map).length = steps.length
        map(steps, f.map).length > Nat.0
        simple_graph_closed_walk(f.dst, f.map(start), map(steps, f.map))
    }
}

/// A graph embedding maps closed walks to closed walks.
theorem simple_graph_embedding_maps_closed_walk[V, W](f: SimpleGraphEmbedding[V, W], start: V, steps: List[V]) {
    simple_graph_closed_walk(f.src, start, steps) implies simple_graph_closed_walk(f.dst, f.map(start), map(steps, f.map))
} by {
    if simple_graph_closed_walk(f.src, start, steps) {
        simple_graph_closed_walk_is_walk(f.src, start, steps)
        simple_graph_closed_walk_positive_length(f.src, start, steps)
        simple_graph_embedding_maps_walk(f, start, start, steps)
        map_length(steps, f.map)
        map(steps, f.map).length = steps.length
        map(steps, f.map).length > Nat.0
        simple_graph_closed_walk(f.dst, f.map(start), map(steps, f.map))
    }
}

/// A graph embedding reflects closed walks whose target list is pointwise mapped from the source.
theorem simple_graph_embedding_reflects_mapped_closed_walk[V, W](
    f: SimpleGraphEmbedding[V, W], start: V, steps: List[V]
) {
    simple_graph_closed_walk(f.dst, f.map(start), map(steps, f.map)) implies
        simple_graph_closed_walk(f.src, start, steps)
} by {
    if simple_graph_closed_walk(f.dst, f.map(start), map(steps, f.map)) {
        simple_graph_closed_walk_is_walk(f.dst, f.map(start), map(steps, f.map))
        simple_graph_walk(f.dst, f.map(start), f.map(start), map(steps, f.map))
        simple_graph_embedding_reflects_mapped_walk(f, start, start, steps)
        simple_graph_walk(f.src, start, start, steps)
        simple_graph_closed_walk_positive_length(f.dst, f.map(start), map(steps, f.map))
        map_length(steps, f.map)
        map(steps, f.map).length = steps.length
        steps.length > Nat.0
        simple_graph_closed_walk(f.src, start, steps)
    }
}
