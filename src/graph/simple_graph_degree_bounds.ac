from nat import Nat, lte_trans
from finite_set import FiniteSet
from data.finite.finite_set_card import fs_card
from data.finite.finite_set_card_compare import fs_card_mono
from data.finite.finite_set_card_bounds import fs_card_le_of_subset_le
from graph.simple_graph import SimpleGraph
from graph.simple_graph_degree import degree, neighborhood, neighborhood_subset
from graph.simple_graph_regular import max_degree_at_most, max_degree_at_most_intro,
    is_regular, is_regular_apply

numerals Nat

/// A vertex has at most as many neighbors as there are vertices.
theorem degree_le_ambient[V](g: SimpleGraph[V], s: FiniteSet[V], v: V) {
    degree(g, s, v) <= fs_card(s)
} by {
    neighborhood_subset(g, s, v)
    neighborhood(g, s, v).subset_eq(s)
    fs_card_mono(neighborhood(g, s, v), s)
    fs_card(neighborhood(g, s, v)) <= fs_card(s)
    degree(g, s, v) <= fs_card(s)
}

/// Every graph obeys the trivial maximum-degree bound.
theorem max_degree_at_most_ambient[V](g: SimpleGraph[V], s: FiniteSet[V]) {
    max_degree_at_most(g, s, fs_card(s))
} by {
    forall(v: V) {
        degree_le_ambient(g, s, v)
        degree(g, s, v) <= fs_card(s)
        s.contains(v) implies degree(g, s, v) <= fs_card(s)
    }
    max_degree_at_most_intro(g, s, fs_card(s))
    max_degree_at_most(g, s, fs_card(s))
}

/// A regular graph's common degree is at most the number of vertices.
theorem regular_degree_le_ambient[V](g: SimpleGraph[V], s: FiniteSet[V], k: Nat, v: V) {
    is_regular(g, s, k) and s.contains(v) implies k <= fs_card(s)
} by {
    if is_regular(g, s, k) and s.contains(v) {
        is_regular_apply(g, s, k, v)
        degree(g, s, v) = k
        degree_le_ambient(g, s, v)
        degree(g, s, v) <= fs_card(s)
        k <= fs_card(s)
    }
}

/// A degree bound on a larger ambient set transfers to any smaller one it contains.
theorem degree_le_of_ambient_bound[V](g: SimpleGraph[V], s: FiniteSet[V], v: V, n: Nat) {
    fs_card(s) <= n implies degree(g, s, v) <= n
} by {
    if fs_card(s) <= n {
        degree_le_ambient(g, s, v)
        degree(g, s, v) <= fs_card(s)
        lte_trans(degree(g, s, v), fs_card(s), n)
        degree(g, s, v) <= n
    }
}

/// The neighborhood of a vertex is bounded by any bound on the ambient set.
theorem neighborhood_card_le[V](g: SimpleGraph[V], s: FiniteSet[V], v: V, n: Nat) {
    fs_card(s) <= n implies fs_card(neighborhood(g, s, v)) <= n
} by {
    if fs_card(s) <= n {
        neighborhood_subset(g, s, v)
        neighborhood(g, s, v).subset_eq(s)
        fs_card_le_of_subset_le(neighborhood(g, s, v), s, n)
        fs_card(neighborhood(g, s, v)) <= n
    }
}
