from data.basic.logic import iff_bool, iff_bool_imp_eq
from data.basic.set import Set
from graph.simple_graph import SimpleGraph, induced_subgraph, induced_subgraph_adj_imp_base_adj,
    graph_intersection, graph_intersection_adj_eq, graph_union, graph_union_adj_eq
from graph.simple_graph_coloring import simple_graph_coloring, simple_graph_coloring_adj_ne

/// Restrict a coloring to an induced subgraph.
theorem simple_graph_coloring_induced_subgraph[V, C](g: SimpleGraph[V], s: Set[V], color: V -> C) {
    simple_graph_coloring(g, color) implies simple_graph_coloring(induced_subgraph(g, s), color)
} by {
    if simple_graph_coloring(g, color) {
        forall(x: V, y: V) {
            if induced_subgraph(g, s).adj(x, y) {
                induced_subgraph_adj_imp_base_adj(g, s, x, y)
                g.adj(x, y)
                simple_graph_coloring_adj_ne(g, color, x, y)
                color(x) != color(y)
            }
        }
    }
}

/// A coloring of the left factor colors the graph intersection.
theorem simple_graph_coloring_graph_intersection_left[V, C](g: SimpleGraph[V], h: SimpleGraph[V], color: V -> C) {
    simple_graph_coloring(g, color) implies simple_graph_coloring(graph_intersection(g, h), color)
} by {
    if simple_graph_coloring(g, color) {
        forall(x: V, y: V) {
            if graph_intersection(g, h).adj(x, y) {
                graph_intersection_adj_eq(g, h, x, y)
                g.adj(x, y)
                simple_graph_coloring_adj_ne(g, color, x, y)
                color(x) != color(y)
            }
        }
    }
}

/// A coloring of the right factor colors the graph intersection.
theorem simple_graph_coloring_graph_intersection_right[V, C](g: SimpleGraph[V], h: SimpleGraph[V], color: V -> C) {
    simple_graph_coloring(h, color) implies simple_graph_coloring(graph_intersection(g, h), color)
} by {
    if simple_graph_coloring(h, color) {
        forall(x: V, y: V) {
            if graph_intersection(g, h).adj(x, y) {
                graph_intersection_adj_eq(g, h, x, y)
                h.adj(x, y)
                simple_graph_coloring_adj_ne(h, color, x, y)
                color(x) != color(y)
            }
        }
    }
}

/// A common coloring of two graphs colors their union.
theorem simple_graph_coloring_graph_union[V, C](g: SimpleGraph[V], h: SimpleGraph[V], color: V -> C) {
    simple_graph_coloring(g, color) and simple_graph_coloring(h, color)
    implies simple_graph_coloring(graph_union(g, h), color)
} by {
    if simple_graph_coloring(g, color) and simple_graph_coloring(h, color) {
        forall(x: V, y: V) {
            if graph_union(g, h).adj(x, y) {
                graph_union_adj_eq(g, h, x, y)
                if g.adj(x, y) {
                    simple_graph_coloring_adj_ne(g, color, x, y)
                    color(x) != color(y)
                } else {
                    h.adj(x, y)
                    simple_graph_coloring_adj_ne(h, color, x, y)
                    color(x) != color(y)
                }
            }
        }
    }
}

/// Union coloring iff both sides are colored.
theorem simple_graph_coloring_graph_union_iff[V, C](g: SimpleGraph[V], h: SimpleGraph[V], color: V -> C) {
    simple_graph_coloring(graph_union(g, h), color) =
    (simple_graph_coloring(g, color) and simple_graph_coloring(h, color))
} by {
    if simple_graph_coloring(graph_union(g, h), color) {
        forall(x: V, y: V) {
            if g.adj(x, y) {
                graph_union_adj_eq(g, h, x, y)
                graph_union(g, h).adj(x, y)
                simple_graph_coloring_adj_ne(graph_union(g, h), color, x, y)
                color(x) != color(y)
            }
        }
        simple_graph_coloring(g, color)
        forall(x: V, y: V) {
            if h.adj(x, y) {
                graph_union_adj_eq(g, h, x, y)
                graph_union(g, h).adj(x, y)
                simple_graph_coloring_adj_ne(graph_union(g, h), color, x, y)
                color(x) != color(y)
            }
        }
        simple_graph_coloring(h, color)
        simple_graph_coloring(g, color) and simple_graph_coloring(h, color)
    }
    if simple_graph_coloring(g, color) and simple_graph_coloring(h, color) {
        simple_graph_coloring_graph_union(g, h, color)
        simple_graph_coloring(graph_union(g, h), color)
    }
    iff_bool(
        simple_graph_coloring(graph_union(g, h), color),
        simple_graph_coloring(g, color) and simple_graph_coloring(h, color)
    )
    iff_bool_imp_eq(
        simple_graph_coloring(graph_union(g, h), color),
        simple_graph_coloring(g, color) and simple_graph_coloring(h, color)
    )
}
