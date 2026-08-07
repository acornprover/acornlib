from nat import Nat
from data.basic.functions import is_injective_fn
from finite_set import FiniteSet, fs_image, finite_set_image_contains_eq,
    finite_set_image_cardinality_is_of_injective
from data.finite.finite_set_membership import finite_set_eq_of_contains_eq
from data.finite.finite_set_card import fs_card, fs_card_eq_of_cardinality_is, fs_card_cardinality_is
from graph.simple_graph import SimpleGraph, is_graph_iso_pair, is_graph_hom,
    is_graph_iso_pair_reflects_adj, is_graph_iso_pair_map_is_injective,
    reflects_graph_adj
from graph.simple_graph_degree import degree, neighborhood, neighborhood_contains_eq

numerals Nat

/// An isomorphism carries adjacency forward.
theorem iso_pair_adj_forward[V, W](
    g: SimpleGraph[V], h: SimpleGraph[W], f: V -> W, e: W -> V, x: V, y: V
) {
    is_graph_iso_pair(g, h, f, e) and g.adj(x, y) implies h.adj(f(x), f(y))
} by {
    if is_graph_iso_pair(g, h, f, e) and g.adj(x, y) {
        is_graph_hom(g, h, f)
        is_graph_hom(g, h, f) = forall(a: V, b: V) {
            g.adj(a, b) implies h.adj(f(a), f(b))
        }
        forall(a: V, b: V) {
            g.adj(a, b) implies h.adj(f(a), f(b))
        }
        g.adj(x, y) implies h.adj(f(x), f(y))
        h.adj(f(x), f(y))
    }
}

/// An isomorphism carries adjacency backward.
theorem iso_pair_adj_backward[V, W](
    g: SimpleGraph[V], h: SimpleGraph[W], f: V -> W, e: W -> V, x: V, y: V
) {
    is_graph_iso_pair(g, h, f, e) and h.adj(f(x), f(y)) implies g.adj(x, y)
} by {
    if is_graph_iso_pair(g, h, f, e) and h.adj(f(x), f(y)) {
        is_graph_iso_pair_reflects_adj(g, h, f, e)
        reflects_graph_adj(g, h, f)
        reflects_graph_adj(g, h, f) = forall(a: V, b: V) {
            h.adj(f(a), f(b)) implies g.adj(a, b)
        }
        forall(a: V, b: V) {
            h.adj(f(a), f(b)) implies g.adj(a, b)
        }
        h.adj(f(x), f(y)) implies g.adj(x, y)
        g.adj(x, y)
    }
}

/// The vertex set carried across an isomorphism.
define mapped_vertices[V, W](s: FiniteSet[V], f: V -> W) -> FiniteSet[W] {
    fs_image(s, f)
}

/// The neighborhood of a mapped vertex in the mapped graph.
///
/// Named so the proofs below work on short terms. Written out, this is
/// `neighborhood(h, fs_image(s, f), f(v))`, which appears often enough in one proof to
/// defeat the assembly.
define mapped_neighborhood[V, W](
    h: SimpleGraph[W], s: FiniteSet[V], f: V -> W, v: V
) -> FiniteSet[W] {
    neighborhood(h, mapped_vertices(s, f), f(v))
}

/// The image of a neighborhood under the isomorphism.
define neighborhood_mapped[V, W](
    g: SimpleGraph[V], s: FiniteSet[V], f: V -> W, v: V
) -> FiniteSet[W] {
    fs_image(neighborhood(g, s, v), f)
}

/// Membership in the mapped neighborhood.
theorem mapped_neighborhood_contains_eq[V, W](
    h: SimpleGraph[W], s: FiniteSet[V], f: V -> W, v: V, y: W
) {
    mapped_neighborhood(h, s, f, v).contains(y) = (mapped_vertices(s, f).contains(y) and h.adj(f(v), y))
} by {
    neighborhood_contains_eq(h, mapped_vertices(s, f), f(v), y)
}

/// Membership in the image of a neighborhood.
theorem neighborhood_mapped_contains_eq[V, W](
    g: SimpleGraph[V], s: FiniteSet[V], f: V -> W, v: V, y: W
) {
    neighborhood_mapped(g, s, f, v).contains(y) = exists(u: V) {
        neighborhood(g, s, v).contains(u) and y = f(u)
    }
} by {
    finite_set_image_contains_eq(neighborhood(g, s, v), f, y)
}

/// Membership in the mapped vertex set.
theorem mapped_vertices_contains_eq[V, W](s: FiniteSet[V], f: V -> W, y: W) {
    mapped_vertices(s, f).contains(y) = exists(u: V) { s.contains(u) and y = f(u) }
} by {
    finite_set_image_contains_eq(s, f, y)
}

/// A member of the mapped neighborhood lies in the image of the neighborhood.
theorem iso_neighborhood_image_forward[V, W](
    g: SimpleGraph[V], h: SimpleGraph[W], f: V -> W, e: W -> V, s: FiniteSet[V], v: V, y: W
) {
    is_graph_iso_pair(g, h, f, e) and mapped_neighborhood(h, s, f, v).contains(y) implies neighborhood_mapped(g, s, f, v).contains(y)
} by {
    if is_graph_iso_pair(g, h, f, e) and mapped_neighborhood(h, s, f, v).contains(y) {
        mapped_neighborhood_contains_eq(h, s, f, v, y)
        mapped_vertices(s, f).contains(y)
        h.adj(f(v), y)
        mapped_vertices_contains_eq(s, f, y)
        let (u: V) satisfy {
            s.contains(u) and y = f(u)
        }
        h.adj(f(v), f(u))
        iso_pair_adj_backward(g, h, f, e, v, u)
        g.adj(v, u)
        neighborhood_contains_eq(g, s, v, u)
        neighborhood(g, s, v).contains(u)
        neighborhood_mapped_contains_eq(g, s, f, v, y)
        neighborhood_mapped(g, s, f, v).contains(y)
    }
}

/// A member of the image of the neighborhood lies in the mapped neighborhood.
theorem iso_neighborhood_image_backward[V, W](
    g: SimpleGraph[V], h: SimpleGraph[W], f: V -> W, e: W -> V, s: FiniteSet[V], v: V, y: W
) {
    is_graph_iso_pair(g, h, f, e) and neighborhood_mapped(g, s, f, v).contains(y) implies mapped_neighborhood(h, s, f, v).contains(y)
} by {
    if is_graph_iso_pair(g, h, f, e) and neighborhood_mapped(g, s, f, v).contains(y) {
        neighborhood_mapped_contains_eq(g, s, f, v, y)
        let (u: V) satisfy {
            neighborhood(g, s, v).contains(u) and y = f(u)
        }
        neighborhood_contains_eq(g, s, v, u)
        s.contains(u)
        g.adj(v, u)
        iso_pair_adj_forward(g, h, f, e, v, u)
        h.adj(f(v), f(u))
        mapped_vertices_contains_eq(s, f, y)
        mapped_vertices(s, f).contains(y)
        mapped_neighborhood_contains_eq(h, s, f, v, y)
        mapped_neighborhood(h, s, f, v).contains(y)
    }
}

/// The two neighborhoods have the same members, pointwise.
///
/// Isolated as its own theorem so the biconditional is a single small goal rather than a
/// step inside a quantified block.
theorem iso_neighborhood_image_iff[V, W](
    g: SimpleGraph[V], h: SimpleGraph[W], f: V -> W, e: W -> V, s: FiniteSet[V], v: V, y: W
) {
    is_graph_iso_pair(g, h, f, e) implies mapped_neighborhood(h, s, f, v).contains(y) = neighborhood_mapped(g, s, f, v).contains(y)
} by {
    if is_graph_iso_pair(g, h, f, e) {
        iso_neighborhood_image_forward(g, h, f, e, s, v, y)
        iso_neighborhood_image_backward(g, h, f, e, s, v, y)
        mapped_neighborhood(h, s, f, v).contains(y) implies neighborhood_mapped(g, s, f, v).contains(y)
        neighborhood_mapped(g, s, f, v).contains(y) implies mapped_neighborhood(h, s, f, v).contains(y)
        mapped_neighborhood(h, s, f, v).contains(y) = neighborhood_mapped(g, s, f, v).contains(y)
    }
}

/// The image of a neighborhood is the neighborhood of the image.
theorem iso_neighborhood_image[V, W](
    g: SimpleGraph[V], h: SimpleGraph[W], f: V -> W, e: W -> V, s: FiniteSet[V], v: V
) {
    is_graph_iso_pair(g, h, f, e) implies mapped_neighborhood(h, s, f, v) = neighborhood_mapped(g, s, f, v)
} by {
    if is_graph_iso_pair(g, h, f, e) {
        forall(y: W) {
            iso_neighborhood_image_iff(g, h, f, e, s, v, y)
            mapped_neighborhood(h, s, f, v).contains(y) = neighborhood_mapped(g, s, f, v).contains(y)
        }
        finite_set_eq_of_contains_eq(mapped_neighborhood(h, s, f, v), neighborhood_mapped(g, s, f, v))
        mapped_neighborhood(h, s, f, v) = neighborhood_mapped(g, s, f, v)
    }
}

/// Degree is invariant under graph isomorphism.
theorem iso_degree_eq[V, W](
    g: SimpleGraph[V], h: SimpleGraph[W], f: V -> W, e: W -> V, s: FiniteSet[V], v: V
) {
    is_graph_iso_pair(g, h, f, e) implies degree(h, mapped_vertices(s, f), f(v)) = degree(g, s, v)
} by {
    if is_graph_iso_pair(g, h, f, e) {
        is_graph_iso_pair_map_is_injective(g, h, f, e)
        is_injective_fn(f)
        fs_card_cardinality_is(neighborhood(g, s, v))
        neighborhood(g, s, v).cardinality_is(fs_card(neighborhood(g, s, v)))
        finite_set_image_cardinality_is_of_injective(
            neighborhood(g, s, v), f, fs_card(neighborhood(g, s, v))
        )
        neighborhood_mapped(g, s, f, v).cardinality_is(fs_card(neighborhood(g, s, v)))
        iso_neighborhood_image(g, h, f, e, s, v)
        mapped_neighborhood(h, s, f, v) = neighborhood_mapped(g, s, f, v)
        mapped_neighborhood(h, s, f, v).cardinality_is(fs_card(neighborhood(g, s, v)))
        fs_card_eq_of_cardinality_is(mapped_neighborhood(h, s, f, v), fs_card(neighborhood(g, s, v)))
        fs_card(mapped_neighborhood(h, s, f, v)) = fs_card(neighborhood(g, s, v))
        degree(h, mapped_vertices(s, f), f(v)) = fs_card(mapped_neighborhood(h, s, f, v))
        degree(g, s, v) = fs_card(neighborhood(g, s, v))
        degree(h, mapped_vertices(s, f), f(v)) = degree(g, s, v)
    }
}
