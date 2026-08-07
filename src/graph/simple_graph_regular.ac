from nat import Nat, lte_trans
from finite_set import FiniteSet
from graph.simple_graph import SimpleGraph
from graph.simple_graph_degree import degree

numerals Nat

/// True if every vertex of `s` has degree at most `n`.
define max_degree_at_most[V](g: SimpleGraph[V], s: FiniteSet[V], n: Nat) -> Bool {
    forall(v: V) {
        s.contains(v) implies degree(g, s, v) <= n
    }
}

/// A vertex of `s` obeys the maximum-degree bound.
theorem max_degree_at_most_apply[V](g: SimpleGraph[V], s: FiniteSet[V], n: Nat, v: V) {
    max_degree_at_most(g, s, n) and s.contains(v) implies degree(g, s, v) <= n
} by {
    if max_degree_at_most(g, s, n) and s.contains(v) {
        max_degree_at_most(g, s, n) = forall(u: V) {
            s.contains(u) implies degree(g, s, u) <= n
        }
        forall(u: V) {
            s.contains(u) implies degree(g, s, u) <= n
        }
        s.contains(v) implies degree(g, s, v) <= n
        degree(g, s, v) <= n
    }
}

/// A pointwise degree bound is a maximum-degree bound.
theorem max_degree_at_most_intro[V](g: SimpleGraph[V], s: FiniteSet[V], n: Nat) {
    (forall(v: V) { s.contains(v) implies degree(g, s, v) <= n })
        implies max_degree_at_most(g, s, n)
} by {
    if forall(v: V) { s.contains(v) implies degree(g, s, v) <= n } {
        max_degree_at_most(g, s, n) = forall(u: V) {
            s.contains(u) implies degree(g, s, u) <= n
        }
    }
}

/// True if every vertex of `s` has degree at least `n`.
define min_degree_at_least[V](g: SimpleGraph[V], s: FiniteSet[V], n: Nat) -> Bool {
    forall(v: V) {
        s.contains(v) implies n <= degree(g, s, v)
    }
}

/// A vertex of `s` obeys the minimum-degree bound.
theorem min_degree_at_least_apply[V](g: SimpleGraph[V], s: FiniteSet[V], n: Nat, v: V) {
    min_degree_at_least(g, s, n) and s.contains(v) implies n <= degree(g, s, v)
} by {
    if min_degree_at_least(g, s, n) and s.contains(v) {
        min_degree_at_least(g, s, n) = forall(u: V) {
            s.contains(u) implies n <= degree(g, s, u)
        }
        forall(u: V) {
            s.contains(u) implies n <= degree(g, s, u)
        }
        s.contains(v) implies n <= degree(g, s, v)
        n <= degree(g, s, v)
    }
}

/// A pointwise lower bound is a minimum-degree bound.
theorem min_degree_at_least_intro[V](g: SimpleGraph[V], s: FiniteSet[V], n: Nat) {
    (forall(v: V) { s.contains(v) implies n <= degree(g, s, v) })
        implies min_degree_at_least(g, s, n)
} by {
    if forall(v: V) { s.contains(v) implies n <= degree(g, s, v) } {
        min_degree_at_least(g, s, n) = forall(u: V) {
            s.contains(u) implies n <= degree(g, s, u)
        }
    }
}

/// True if every vertex of `s` has degree exactly `k`.
define is_regular[V](g: SimpleGraph[V], s: FiniteSet[V], k: Nat) -> Bool {
    forall(v: V) {
        s.contains(v) implies degree(g, s, v) = k
    }
}

/// A vertex of a regular graph has the common degree.
theorem is_regular_apply[V](g: SimpleGraph[V], s: FiniteSet[V], k: Nat, v: V) {
    is_regular(g, s, k) and s.contains(v) implies degree(g, s, v) = k
} by {
    if is_regular(g, s, k) and s.contains(v) {
        is_regular(g, s, k) = forall(u: V) {
            s.contains(u) implies degree(g, s, u) = k
        }
        forall(u: V) {
            s.contains(u) implies degree(g, s, u) = k
        }
        s.contains(v) implies degree(g, s, v) = k
        degree(g, s, v) = k
    }
}

/// A pointwise common degree is regularity.
theorem is_regular_intro[V](g: SimpleGraph[V], s: FiniteSet[V], k: Nat) {
    (forall(v: V) { s.contains(v) implies degree(g, s, v) = k }) implies is_regular(g, s, k)
} by {
    if forall(v: V) { s.contains(v) implies degree(g, s, v) = k } {
        is_regular(g, s, k) = forall(u: V) {
            s.contains(u) implies degree(g, s, u) = k
        }
    }
}

/// True if every vertex of `s` has degree exactly three.
define is_cubic[V](g: SimpleGraph[V], s: FiniteSet[V]) -> Bool {
    is_regular(g, s, Nat.3)
}

/// A cubic graph is three-regular.
theorem cubic_is_regular_three[V](g: SimpleGraph[V], s: FiniteSet[V]) {
    is_cubic(g, s) implies is_regular(g, s, Nat.3)
}

/// A three-regular graph is cubic.
theorem cubic_of_regular_three[V](g: SimpleGraph[V], s: FiniteSet[V]) {
    is_regular(g, s, Nat.3) implies is_cubic(g, s)
}

/// A vertex of a cubic graph has degree three.
theorem cubic_degree[V](g: SimpleGraph[V], s: FiniteSet[V], v: V) {
    is_cubic(g, s) and s.contains(v) implies degree(g, s, v) = Nat.3
} by {
    if is_cubic(g, s) and s.contains(v) {
        is_regular(g, s, Nat.3)
        is_regular_apply(g, s, Nat.3, v)
    }
}

/// A regular graph meets the matching upper bound.
theorem regular_max_degree_at_most[V](g: SimpleGraph[V], s: FiniteSet[V], k: Nat) {
    is_regular(g, s, k) implies max_degree_at_most(g, s, k)
} by {
    if is_regular(g, s, k) {
        forall(v: V) {
            if s.contains(v) {
                is_regular_apply(g, s, k, v)
                degree(g, s, v) = k
                degree(g, s, v) <= k
            }
            s.contains(v) implies degree(g, s, v) <= k
        }
        max_degree_at_most_intro(g, s, k)
        max_degree_at_most(g, s, k)
    }
}

/// A regular graph meets the matching lower bound.
theorem regular_min_degree_at_least[V](g: SimpleGraph[V], s: FiniteSet[V], k: Nat) {
    is_regular(g, s, k) implies min_degree_at_least(g, s, k)
} by {
    if is_regular(g, s, k) {
        forall(v: V) {
            if s.contains(v) {
                is_regular_apply(g, s, k, v)
                degree(g, s, v) = k
                k <= degree(g, s, v)
            }
            s.contains(v) implies k <= degree(g, s, v)
        }
        min_degree_at_least_intro(g, s, k)
        min_degree_at_least(g, s, k)
    }
}

/// Matching upper and lower degree bounds force regularity.
theorem regular_of_bounds[V](g: SimpleGraph[V], s: FiniteSet[V], k: Nat) {
    max_degree_at_most(g, s, k) and min_degree_at_least(g, s, k) implies is_regular(g, s, k)
} by {
    if max_degree_at_most(g, s, k) and min_degree_at_least(g, s, k) {
        forall(v: V) {
            if s.contains(v) {
                max_degree_at_most_apply(g, s, k, v)
                min_degree_at_least_apply(g, s, k, v)
                degree(g, s, v) <= k
                k <= degree(g, s, v)
                degree(g, s, v) = k
            }
            s.contains(v) implies degree(g, s, v) = k
        }
        is_regular_intro(g, s, k)
        is_regular(g, s, k)
    }
}

/// Maximum-degree bounds weaken upward.
theorem max_degree_at_most_weaken[V](g: SimpleGraph[V], s: FiniteSet[V], m: Nat, n: Nat) {
    max_degree_at_most(g, s, m) and m <= n implies max_degree_at_most(g, s, n)
} by {
    if max_degree_at_most(g, s, m) and m <= n {
        forall(v: V) {
            if s.contains(v) {
                max_degree_at_most_apply(g, s, m, v)
                degree(g, s, v) <= m
                lte_trans(degree(g, s, v), m, n)
                degree(g, s, v) <= n
            }
            s.contains(v) implies degree(g, s, v) <= n
        }
        max_degree_at_most_intro(g, s, n)
        max_degree_at_most(g, s, n)
    }
}

/// Minimum-degree bounds weaken downward.
theorem min_degree_at_least_weaken[V](g: SimpleGraph[V], s: FiniteSet[V], m: Nat, n: Nat) {
    min_degree_at_least(g, s, m) and n <= m implies min_degree_at_least(g, s, n)
} by {
    if min_degree_at_least(g, s, m) and n <= m {
        forall(v: V) {
            if s.contains(v) {
                min_degree_at_least_apply(g, s, m, v)
                m <= degree(g, s, v)
                lte_trans(n, m, degree(g, s, v))
                n <= degree(g, s, v)
            }
            s.contains(v) implies n <= degree(g, s, v)
        }
        min_degree_at_least_intro(g, s, n)
        min_degree_at_least(g, s, n)
    }
}

/// Every finite vertex set has degree at least zero.
theorem min_degree_at_least_zero[V](g: SimpleGraph[V], s: FiniteSet[V]) {
    min_degree_at_least(g, s, Nat.0)
} by {
    forall(v: V) {
        Nat.0 <= degree(g, s, v)
        s.contains(v) implies Nat.0 <= degree(g, s, v)
    }
    min_degree_at_least_intro(g, s, Nat.0)
    min_degree_at_least(g, s, Nat.0)
}

/// A cubic graph has maximum degree three.
theorem cubic_max_degree_three[V](g: SimpleGraph[V], s: FiniteSet[V]) {
    is_cubic(g, s) implies max_degree_at_most(g, s, Nat.3)
} by {
    if is_cubic(g, s) {
        is_regular(g, s, Nat.3)
        regular_max_degree_at_most(g, s, Nat.3)
    }
}

/// A cubic graph has minimum degree three.
theorem cubic_min_degree_three[V](g: SimpleGraph[V], s: FiniteSet[V]) {
    is_cubic(g, s) implies min_degree_at_least(g, s, Nat.3)
} by {
    if is_cubic(g, s) {
        is_regular(g, s, Nat.3)
        regular_min_degree_at_least(g, s, Nat.3)
    }
}
