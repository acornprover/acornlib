from nat import Nat, lte_antisymm
from finite_set import FiniteSet, finite_set_empty_subset
from data.finite.finite_set_card import fs_card, fs_card_empty
from data.finite.finite_set_card_compare import fs_card_mono
from graph.simple_graph import SimpleGraph, graph_complement
from graph.simple_graph_independent import is_clique_in, is_independent_in
from graph.simple_graph_independent_domination import is_independent_subset,
    is_independent_subset_apply, is_independent_subset_intro
from graph.simple_graph_max_independent import independence_number, independence_number_attained,
    independence_number_is_greatest, independent_size_pred, maximum_independent_set_exists
from graph.simple_graph_complement_duality import clique_is_independent_in_complement,
    independent_in_complement_is_clique, independent_is_clique_in_complement,
    clique_in_complement_is_independent
from data.nat.nat_bounded_max import is_max, has_max, is_max_apply, is_max_intro,
    is_max_is_upper_bound, is_upper_bound_of, is_upper_bound_of_intro, max_unique

numerals Nat

/// A subset of `s` that is a clique.
define is_clique_subset[V](
    g: SimpleGraph[V], s: FiniteSet[V], t: FiniteSet[V]
) -> Bool {
    t.subset_eq(s) and is_clique_in(g, t)
}

/// A clique subset is a subset that is a clique.
theorem is_clique_subset_apply[V](
    g: SimpleGraph[V], s: FiniteSet[V], t: FiniteSet[V]
) {
    is_clique_subset(g, s, t) implies t.subset_eq(s) and is_clique_in(g, t)
} by {
    if is_clique_subset(g, s, t) {
        (is_clique_subset(g, s, t) = (t.subset_eq(s) and is_clique_in(g, t)))
        (t.subset_eq(s) and is_clique_in(g, t))
    }
}

/// A subset that is a clique is a clique subset.
theorem is_clique_subset_intro[V](
    g: SimpleGraph[V], s: FiniteSet[V], t: FiniteSet[V]
) {
    t.subset_eq(s) and is_clique_in(g, t) implies is_clique_subset(g, s, t)
} by {
    if t.subset_eq(s) and is_clique_in(g, t) {
        (is_clique_subset(g, s, t) = (t.subset_eq(s) and is_clique_in(g, t)))
        is_clique_subset(g, s, t)
    }
}

/// Clique subsets of a graph are exactly independent subsets of its complement.
theorem clique_subset_iff_independent_subset_complement[V](
    g: SimpleGraph[V], s: FiniteSet[V], t: FiniteSet[V]
) {
    is_clique_subset(g, s, t) = is_independent_subset(graph_complement(g), s, t)
} by {
    if is_clique_subset(g, s, t) {
        is_clique_subset_apply(g, s, t)
        (t.subset_eq(s) and is_clique_in(g, t))
        clique_is_independent_in_complement(g, t)
        is_independent_in(graph_complement(g), t)
        is_independent_subset_intro(graph_complement(g), s, t)
        is_independent_subset(graph_complement(g), s, t)
    }
    if is_independent_subset(graph_complement(g), s, t) {
        is_independent_subset_apply(graph_complement(g), s, t)
        (t.subset_eq(s) and is_independent_in(graph_complement(g), t))
        independent_in_complement_is_clique(g, t)
        is_clique_in(g, t)
        is_clique_subset_intro(g, s, t)
        is_clique_subset(g, s, t)
    }
    (is_clique_subset(g, s, t) implies is_independent_subset(graph_complement(g), s, t))
    (is_independent_subset(graph_complement(g), s, t) implies is_clique_subset(g, s, t))
    is_clique_subset(g, s, t) = is_independent_subset(graph_complement(g), s, t)
}

/// True of the sizes achieved by clique subsets of `s`.
define clique_size_pred[V](g: SimpleGraph[V], s: FiniteSet[V]) -> (Nat -> Bool) {
    function(n: Nat) {
        exists(t: FiniteSet[V]) {
            is_clique_subset(g, s, t) and fs_card(t) = n
        }
    }
}

/// A clique subset achieves its own size.
theorem clique_size_pred_intro[V](
    g: SimpleGraph[V], s: FiniteSet[V], t: FiniteSet[V]
) {
    is_clique_subset(g, s, t) implies clique_size_pred(g, s)(fs_card(t))
} by {
    if is_clique_subset(g, s, t) {
        (clique_size_pred(g, s)(fs_card(t)) = exists(r: FiniteSet[V]) {
            is_clique_subset(g, s, r) and fs_card(r) = fs_card(t)
        })
        exists(r: FiniteSet[V]) {
            is_clique_subset(g, s, r) and fs_card(r) = fs_card(t)
        }
        clique_size_pred(g, s)(fs_card(t))
    }
}

/// An achieved clique size comes from a clique subset.
theorem clique_size_pred_witness[V](g: SimpleGraph[V], s: FiniteSet[V], n: Nat) {
    clique_size_pred(g, s)(n) implies exists(t: FiniteSet[V]) {
        is_clique_subset(g, s, t) and fs_card(t) = n
    }
} by {
    if clique_size_pred(g, s)(n) {
        (clique_size_pred(g, s)(n) = exists(t: FiniteSet[V]) {
            is_clique_subset(g, s, t) and fs_card(t) = n
        })
        exists(t: FiniteSet[V]) {
            is_clique_subset(g, s, t) and fs_card(t) = n
        }
    }
}

/// The independence number of the complement is an achieved clique size.
theorem clique_size_pred_independence_number[V](g: SimpleGraph[V], s: FiniteSet[V]) {
    clique_size_pred(g, s)(independence_number(graph_complement(g), s))
} by {
    maximum_independent_set_exists(graph_complement(g), s)
    exists(t: FiniteSet[V]) {
        is_independent_subset(graph_complement(g), s, t)
            and fs_card(t) = independence_number(graph_complement(g), s)
    }
    let (t: FiniteSet[V]) satisfy {
        is_independent_subset(graph_complement(g), s, t)
            and fs_card(t) = independence_number(graph_complement(g), s)
    }
    clique_subset_iff_independent_subset_complement(g, s, t)
    is_clique_subset(g, s, t)
    clique_size_pred_intro(g, s, t)
    clique_size_pred(g, s)(fs_card(t))
    clique_size_pred(g, s)(independence_number(graph_complement(g), s))
}

/// The independence number of the complement bounds every clique size.
theorem clique_size_pred_independence_bound[V](g: SimpleGraph[V], s: FiniteSet[V]) {
    is_upper_bound_of(clique_size_pred(g, s), independence_number(graph_complement(g), s))
} by {
    forall(n: Nat) {
        if clique_size_pred(g, s)(n) {
            clique_size_pred_witness(g, s, n)
            exists(t: FiniteSet[V]) {
                is_clique_subset(g, s, t) and fs_card(t) = n
            }
            let (t: FiniteSet[V]) satisfy {
                is_clique_subset(g, s, t) and fs_card(t) = n
            }
            clique_subset_iff_independent_subset_complement(g, s, t)
            is_independent_subset(graph_complement(g), s, t)
            independence_number_is_greatest(graph_complement(g), s, t)
            fs_card(t) <= independence_number(graph_complement(g), s)
            n <= independence_number(graph_complement(g), s)
        }
        (clique_size_pred(g, s)(n)
            implies n <= independence_number(graph_complement(g), s))
    }
    is_upper_bound_of_intro(clique_size_pred(g, s),
        independence_number(graph_complement(g), s))
    is_upper_bound_of(clique_size_pred(g, s), independence_number(graph_complement(g), s))
}

/// The largest size of a clique subset of `s`.
///
/// The clique number. Well defined for the same reason the independence number is: the empty
/// set is a clique and every clique subset is bounded by the ambient set. Here the maximum is
/// obtained from the independence number of the complement, which the duality already shows to
/// be one.
let clique_number[V](g: SimpleGraph[V], s: FiniteSet[V]) -> result: Nat satisfy {
    is_max(clique_size_pred(g, s), result)
} by {
    clique_size_pred_independence_number(g, s)
    clique_size_pred_independence_bound(g, s)
    is_max_intro(clique_size_pred(g, s), independence_number(graph_complement(g), s))
    is_max(clique_size_pred(g, s), independence_number(graph_complement(g), s))
}

/// Some clique subset attains the clique number.
theorem clique_number_attained[V](g: SimpleGraph[V], s: FiniteSet[V]) {
    clique_size_pred(g, s)(clique_number(g, s))
} by {
    is_max(clique_size_pred(g, s), clique_number(g, s))
    is_max_apply(clique_size_pred(g, s), clique_number(g, s))
}

/// No clique subset is larger than the clique number.
theorem clique_number_is_greatest[V](
    g: SimpleGraph[V], s: FiniteSet[V], t: FiniteSet[V]
) {
    is_clique_subset(g, s, t) implies fs_card(t) <= clique_number(g, s)
} by {
    if is_clique_subset(g, s, t) {
        clique_size_pred_intro(g, s, t)
        clique_size_pred(g, s)(fs_card(t))
        is_max(clique_size_pred(g, s), clique_number(g, s))
        is_max_is_upper_bound(clique_size_pred(g, s), clique_number(g, s), fs_card(t))
        fs_card(t) <= clique_number(g, s)
    }
}

/// The clique number of a graph is the independence number of its complement.
///
/// The two maximisations run over the same family of subsets, by the duality, so they have the
/// same maximum. This is the invariant form of the complement duality.
theorem clique_number_eq_complement_independence_number[V](
    g: SimpleGraph[V], s: FiniteSet[V]
) {
    clique_number(g, s) = independence_number(graph_complement(g), s)
} by {
    is_max(clique_size_pred(g, s), clique_number(g, s))
    clique_size_pred_independence_number(g, s)
    clique_size_pred_independence_bound(g, s)
    is_max_intro(clique_size_pred(g, s), independence_number(graph_complement(g), s))
    is_max(clique_size_pred(g, s), independence_number(graph_complement(g), s))
    max_unique(clique_size_pred(g, s), clique_number(g, s),
        independence_number(graph_complement(g), s))
    clique_number(g, s) = independence_number(graph_complement(g), s)
}

/// Independent subsets of a graph are exactly clique subsets of its complement.
theorem independent_subset_iff_clique_subset_complement[V](
    g: SimpleGraph[V], s: FiniteSet[V], t: FiniteSet[V]
) {
    is_independent_subset(g, s, t) = is_clique_subset(graph_complement(g), s, t)
} by {
    if is_independent_subset(g, s, t) {
        is_independent_subset_apply(g, s, t)
        (t.subset_eq(s) and is_independent_in(g, t))
        independent_is_clique_in_complement(g, t)
        is_clique_in(graph_complement(g), t)
        is_clique_subset_intro(graph_complement(g), s, t)
        is_clique_subset(graph_complement(g), s, t)
    }
    if is_clique_subset(graph_complement(g), s, t) {
        is_clique_subset_apply(graph_complement(g), s, t)
        (t.subset_eq(s) and is_clique_in(graph_complement(g), t))
        clique_in_complement_is_independent(g, t)
        is_independent_in(g, t)
        is_independent_subset_intro(g, s, t)
        is_independent_subset(g, s, t)
    }
    (is_independent_subset(g, s, t) implies is_clique_subset(graph_complement(g), s, t))
    (is_clique_subset(graph_complement(g), s, t) implies is_independent_subset(g, s, t))
    is_independent_subset(g, s, t) = is_clique_subset(graph_complement(g), s, t)
}

/// The independence number of a graph is the clique number of its complement.
///
/// The other half of the invariant duality. Proved the same way rather than through an
/// involution law for the complement, which the library does not have.
theorem independence_number_eq_complement_clique_number[V](
    g: SimpleGraph[V], s: FiniteSet[V]
) {
    independence_number(g, s) = clique_number(graph_complement(g), s)
} by {
    is_max(independent_size_pred(g, s), independence_number(g, s))
    clique_number_attained(graph_complement(g), s)
    clique_size_pred(graph_complement(g), s)(clique_number(graph_complement(g), s))
    clique_size_pred_witness(graph_complement(g), s, clique_number(graph_complement(g), s))
    exists(t: FiniteSet[V]) {
        is_clique_subset(graph_complement(g), s, t)
            and fs_card(t) = clique_number(graph_complement(g), s)
    }
    let (t: FiniteSet[V]) satisfy {
        is_clique_subset(graph_complement(g), s, t)
            and fs_card(t) = clique_number(graph_complement(g), s)
    }
    independent_subset_iff_clique_subset_complement(g, s, t)
    is_independent_subset(g, s, t)
    independence_number_is_greatest(g, s, t)
    fs_card(t) <= independence_number(g, s)
    clique_number(graph_complement(g), s) <= independence_number(g, s)
    maximum_independent_set_exists(g, s)
    exists(r: FiniteSet[V]) {
        is_independent_subset(g, s, r) and fs_card(r) = independence_number(g, s)
    }
    let (r: FiniteSet[V]) satisfy {
        is_independent_subset(g, s, r) and fs_card(r) = independence_number(g, s)
    }
    independent_subset_iff_clique_subset_complement(g, s, r)
    is_clique_subset(graph_complement(g), s, r)
    clique_number_is_greatest(graph_complement(g), s, r)
    fs_card(r) <= clique_number(graph_complement(g), s)
    independence_number(g, s) <= clique_number(graph_complement(g), s)
    lte_antisymm(independence_number(g, s), clique_number(graph_complement(g), s))
    independence_number(g, s) = clique_number(graph_complement(g), s)
}
