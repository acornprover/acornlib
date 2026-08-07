from nat import Nat
from finite_set import FiniteSet
from data.finite.finite_set_card import fs_card, fs_card_eq_of_cardinality_is
from data.finite.finite_set_filter import finite_set_filter, finite_set_filter_contains_eq,
    finite_set_filter_subset
from graph.simple_graph import SimpleGraph, empty_graph, empty_graph_adj, complete_graph,
    complete_graph_adj, simple_graph_adj_irreflexive, simple_graph_adj_comm
from data.basic.set import Set

numerals Nat

/// True of the vertices adjacent to `v` in `g`.
///
/// Packaged as a predicate so it can index a finite set.
define neighbor_pred[V](g: SimpleGraph[V], v: V) -> (V -> Bool) {
    function(w: V) {
        g.adj(v, w)
    }
}

/// The neighbors of `v` lying in the finite vertex set `s`.
define neighborhood[V](g: SimpleGraph[V], s: FiniteSet[V], v: V) -> FiniteSet[V] {
    finite_set_filter(s, neighbor_pred(g, v))
}

/// A vertex belongs to a neighborhood exactly when it is in the ambient set and adjacent.
theorem neighborhood_contains_eq[V](g: SimpleGraph[V], s: FiniteSet[V], v: V, w: V) {
    neighborhood(g, s, v).contains(w) = (s.contains(w) and g.adj(v, w))
} by {
    finite_set_filter_contains_eq(s, neighbor_pred(g, v), w)
    neighbor_pred(g, v)(w) = g.adj(v, w)
}

/// Neighborhoods sit inside the ambient vertex set.
theorem neighborhood_subset[V](g: SimpleGraph[V], s: FiniteSet[V], v: V) {
    neighborhood(g, s, v).subset_eq(s)
} by {
    finite_set_filter_subset(s, neighbor_pred(g, v))
}

/// No vertex is its own neighbor.
theorem neighborhood_not_contains_self[V](g: SimpleGraph[V], s: FiniteSet[V], v: V) {
    not neighborhood(g, s, v).contains(v)
} by {
    neighborhood_contains_eq(g, s, v, v)
    simple_graph_adj_irreflexive(g, v)
    not g.adj(v, v)
}

/// Neighborhood membership is symmetric in the two vertices, given the ambient set contains both.
theorem neighborhood_contains_comm[V](g: SimpleGraph[V], s: FiniteSet[V], v: V, w: V) {
    s.contains(v) and s.contains(w) implies
        neighborhood(g, s, v).contains(w) = neighborhood(g, s, w).contains(v)
} by {
    if s.contains(v) and s.contains(w) {
        neighborhood_contains_eq(g, s, v, w)
        neighborhood_contains_eq(g, s, w, v)
        simple_graph_adj_comm(g, v, w)
    }
}

/// The degree of `v` in `g`, counted within the finite vertex set `s`.
define degree[V](g: SimpleGraph[V], s: FiniteSet[V], v: V) -> Nat {
    fs_card(neighborhood(g, s, v))
}

/// Degree is the size of the neighborhood.
theorem degree_eq_neighborhood_card[V](g: SimpleGraph[V], s: FiniteSet[V], v: V) {
    degree(g, s, v) = fs_card(neighborhood(g, s, v))
}

/// A vertex with a neighbor in the ambient set has a nonempty neighborhood.
theorem neighborhood_contains_of_adj[V](g: SimpleGraph[V], s: FiniteSet[V], v: V, w: V) {
    s.contains(w) and g.adj(v, w) implies neighborhood(g, s, v).contains(w)
} by {
    if s.contains(w) and g.adj(v, w) {
        neighborhood_contains_eq(g, s, v, w)
    }
}

/// In the empty graph no vertex has neighbors.
theorem empty_graph_neighborhood_is_empty[V](s: FiniteSet[V], v: V, w: V) {
    not neighborhood(empty_graph[V], s, v).contains(w)
} by {
    neighborhood_contains_eq(empty_graph[V], s, v, w)
    empty_graph[V].adj(v, w) = empty_graph_adj(v, w)
    not empty_graph_adj(v, w)
}

/// In the complete graph the neighbors of `v` are the other vertices of the ambient set.
theorem complete_graph_neighborhood_contains_eq[V](s: FiniteSet[V], v: V, w: V) {
    neighborhood(complete_graph[V], s, v).contains(w) = (s.contains(w) and v != w)
} by {
    neighborhood_contains_eq(complete_graph[V], s, v, w)
    complete_graph[V].adj(v, w) = complete_graph_adj(v, w)
    complete_graph_adj(v, w) = (v != w)
}
