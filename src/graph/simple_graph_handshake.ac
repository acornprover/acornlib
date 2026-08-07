from nat import Nat
from pair import Pair
from finite_set import FiniteSet, fs_union, fs_insert, finite_set_sum,
    finite_set_sum_empty, finite_set_sum_insert, finite_set_subset_refl,
    finite_set_subset_contains
from data.finite.finite_set_card import fs_card
from data.finite.finite_set_membership import fs_insert_contains_self, fs_insert_contains_of_contains
from data.finite.finite_set_subset_intro import fs_subset_eq_intro
from data.finite.finite_set_unique_induction import finite_set_strong_induction
from graph.simple_graph import SimpleGraph
from graph.simple_graph_degree import degree
from graph.simple_graph_degree_sum import degree_fn, degree_fn_eq, degree_sum,
    directed_edges_from
from graph.simple_graph_edges_from_card import directed_edges_from_card
from graph.simple_graph_edge_fibers import directed_edges_from_set, fs_card_disjoint_union,
    directed_edges_from_disjoint
from graph.simple_graph_fiber_split import directed_edges_from_set_insert,
    directed_edges_from_set_empty_card

numerals Nat

/// True when the degree sum over `t` matches the count of edges leaving `t`.
///
/// The ambient set `s` is fixed and only `t` varies, so every degree stays constant
/// through the induction.
define handshake_holds[V](
    g: SimpleGraph[V], s: FiniteSet[V], t: FiniteSet[V]
) -> Bool {
    t.subset_eq(s) implies finite_set_sum(t, degree_fn(g, s)) = fs_card(directed_edges_from_set(g, s, t))
}

/// The empty index set contributes nothing on either side.
theorem handshake_holds_empty[V](g: SimpleGraph[V], s: FiniteSet[V]) {
    handshake_holds(g, s, FiniteSet.empty[V])
} by {
    if FiniteSet.empty[V].subset_eq(s) {
        finite_set_sum_empty(degree_fn(g, s))
        finite_set_sum(FiniteSet.empty[V], degree_fn(g, s)) = Nat.0
        directed_edges_from_set_empty_card(g, s)
        fs_card(directed_edges_from_set(g, s, FiniteSet.empty[V])) = Nat.0
        finite_set_sum(FiniteSet.empty[V], degree_fn(g, s)) = fs_card(directed_edges_from_set(g, s, FiniteSet.empty[V]))
    }
    handshake_holds(g, s, FiniteSet.empty[V])
}

/// Adding a fresh vertex adds its degree on one side and its outgoing edges on the other.
theorem handshake_holds_insert[V](
    g: SimpleGraph[V], s: FiniteSet[V], t: FiniteSet[V], v: V
) {
    handshake_holds(g, s, t) and not t.contains(v) implies handshake_holds(g, s, fs_insert(t, v))
} by {
    if handshake_holds(g, s, t) and not t.contains(v) {
        if fs_insert(t, v).subset_eq(s) {
            fs_insert_contains_self(t, v)
            fs_insert(t, v).contains(v)
            finite_set_subset_contains(fs_insert(t, v), s, v)
            s.contains(v)
            forall(x: V) {
                if t.contains(x) {
                    fs_insert_contains_of_contains(t, v, x)
                    fs_insert(t, v).contains(x)
                    finite_set_subset_contains(fs_insert(t, v), s, x)
                    s.contains(x)
                }
                t.contains(x) implies s.contains(x)
            }
            fs_subset_eq_intro(t, s)
            t.subset_eq(s)
            finite_set_sum(t, degree_fn(g, s)) = fs_card(directed_edges_from_set(g, s, t))
            finite_set_sum_insert(t, v, degree_fn(g, s))
            finite_set_sum(fs_insert(t, v), degree_fn(g, s)) = degree_fn(g, s)(v) + finite_set_sum(t, degree_fn(g, s))
            degree_fn_eq(g, s, v)
            degree_fn(g, s)(v) = degree(g, s, v)
            directed_edges_from_card(g, s, v)
            fs_card(directed_edges_from(g, s, v)) = degree(g, s, v)
            directed_edges_from_disjoint(g, s, t, v)
            directed_edges_from(g, s, v).is_disjoint(directed_edges_from_set(g, s, t))
            fs_card_disjoint_union(directed_edges_from(g, s, v), directed_edges_from_set(g, s, t))
            fs_card(fs_union(directed_edges_from(g, s, v), directed_edges_from_set(g, s, t))) = fs_card(directed_edges_from(g, s, v)) + fs_card(directed_edges_from_set(g, s, t))
            directed_edges_from_set_insert(g, s, t, v)
            directed_edges_from_set(g, s, fs_insert(t, v)) = fs_union(directed_edges_from(g, s, v), directed_edges_from_set(g, s, t))
            fs_card(directed_edges_from_set(g, s, fs_insert(t, v))) = degree(g, s, v) + fs_card(directed_edges_from_set(g, s, t))
            finite_set_sum(fs_insert(t, v), degree_fn(g, s)) = fs_card(directed_edges_from_set(g, s, fs_insert(t, v)))
        }
        handshake_holds(g, s, fs_insert(t, v))
    }
}

/// The handshake lemma.
///
/// The sum of the degrees of the vertices equals the number of ordered adjacent pairs,
/// which is twice the number of edges since each edge is counted in both orientations.
///
/// Proved by induction on a subset of the vertex set, holding the ambient set fixed so
/// that no degree changes during the induction. The step uses that the edges leaving a
/// fresh vertex are disjoint from those leaving the vertices already counted.
theorem handshake_lemma[V](g: SimpleGraph[V], s: FiniteSet[V]) {
    degree_sum(g, s) = fs_card(directed_edges_from_set(g, s, s))
} by {
    handshake_holds_empty(g, s)
    forall(t: FiniteSet[V], item: V) {
        handshake_holds_insert(g, s, t, item)
        handshake_holds(g, s, t) and not t.contains(item) implies handshake_holds(g, s, fs_insert(t, item))
    }
    finite_set_strong_induction(handshake_holds(g, s), s)
    handshake_holds(g, s, s)
    finite_set_subset_refl(s)
    s.subset_eq(s)
    finite_set_sum(s, degree_fn(g, s)) = fs_card(directed_edges_from_set(g, s, s))
    degree_sum(g, s) = finite_set_sum(s, degree_fn(g, s))
}
