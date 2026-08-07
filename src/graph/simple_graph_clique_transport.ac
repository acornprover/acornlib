from data.basic.set import Set, set_image_contains_eq
from graph.simple_graph import SimpleGraphHom, is_clique, is_clique_contains_adj, is_independent_set,
    is_independent_set_contains_not_adj, simple_graph_adj_ne, simple_graph_hom_image_set,
    simple_graph_hom_image_set_eq, simple_graph_hom_maps_adj, simple_graph_hom_preimage_set,
    simple_graph_hom_preimage_set_contains_eq

/// The image of a clique under a graph homomorphism is a clique in the target graph.
theorem simple_graph_hom_image_clique[V, W](f: SimpleGraphHom[V, W], s: Set[V]) {
    is_clique(f.src, s) implies is_clique(f.dst, simple_graph_hom_image_set(f, s))
} by {
    if is_clique(f.src, s) {
        is_clique(f.src, s) = forall(x: V, y: V) {
            s.contains(x) and s.contains(y) and x != y implies f.src.adj(x, y)
        }
        simple_graph_hom_image_set_eq(f, s)
        forall(y1: W, y2: W) {
            if simple_graph_hom_image_set(f, s).contains(y1)
                and simple_graph_hom_image_set(f, s).contains(y2) and y1 != y2 {
                set_image_contains_eq(s, f.map, y1)
                let x1: V satisfy {
                    s.contains(x1) and y1 = f.map(x1)
                }
                set_image_contains_eq(s, f.map, y2)
                let x2: V satisfy {
                    s.contains(x2) and y2 = f.map(x2)
                }
                if x1 = x2 {
                    y1 = f.map(x2)
                    y2 = f.map(x2)
                    y1 = y2
                    false
                }
                x1 != x2
                is_clique(f.src, s) and s.contains(x1) and s.contains(x2) and x1 != x2
                is_clique_contains_adj(f.src, s, x1, x2)
                f.src.adj(x1, x2)
                simple_graph_hom_maps_adj(f, x1, x2)
                f.dst.adj(f.map(x1), f.map(x2))
                f.dst.adj(y1, y2)
            }
        }
        is_clique(f.dst, simple_graph_hom_image_set(f, s)) = forall(y1: W, y2: W) {
            simple_graph_hom_image_set(f, s).contains(y1)
                and simple_graph_hom_image_set(f, s).contains(y2) and y1 != y2
                implies f.dst.adj(y1, y2)
        }
        if not is_clique(f.dst, simple_graph_hom_image_set(f, s)) {
            let bad_y1: W satisfy {
                exists(bad_y2: W) {
                    simple_graph_hom_image_set(f, s).contains(bad_y1)
                        and simple_graph_hom_image_set(f, s).contains(bad_y2) and bad_y1 != bad_y2
                        and not f.dst.adj(bad_y1, bad_y2)
                }
            }
            let bad_y2: W satisfy {
                simple_graph_hom_image_set(f, s).contains(bad_y1)
                    and simple_graph_hom_image_set(f, s).contains(bad_y2) and bad_y1 != bad_y2
                    and not f.dst.adj(bad_y1, bad_y2)
            }
            simple_graph_hom_image_set(f, s).contains(bad_y1) and
                simple_graph_hom_image_set(f, s).contains(bad_y2) and bad_y1 != bad_y2
            simple_graph_hom_image_set(f, s).contains(bad_y2) and bad_y1 != bad_y2
            simple_graph_hom_image_set(f, s).contains(bad_y1)
            simple_graph_hom_image_set(f, s).contains(bad_y2)
            set_image_contains_eq(s, f.map, bad_y1)
            let bad_x1: V satisfy {
                s.contains(bad_x1) and bad_y1 = f.map(bad_x1)
            }
            set_image_contains_eq(s, f.map, bad_y2)
            let bad_x2: V satisfy {
                s.contains(bad_x2) and bad_y2 = f.map(bad_x2)
            }
            if bad_x1 = bad_x2 {
                bad_y1 = f.map(bad_x2)
                bad_y2 = f.map(bad_x2)
                bad_y1 = bad_y2
                false
            }
            bad_x1 != bad_x2
            is_clique(f.src, s) and s.contains(bad_x1) and s.contains(bad_x2) and bad_x1 != bad_x2
            is_clique_contains_adj(f.src, s, bad_x1, bad_x2)
            f.src.adj(bad_x1, bad_x2)
            simple_graph_hom_maps_adj(f, bad_x1, bad_x2)
            f.dst.adj(f.map(bad_x1), f.map(bad_x2))
            f.dst.adj(bad_y1, bad_y2)
            false
        }
        is_clique(f.dst, simple_graph_hom_image_set(f, s))
    }
}

/// The preimage of an independent set under a graph homomorphism is independent in the source graph.
theorem simple_graph_hom_preimage_independent_set[V, W](f: SimpleGraphHom[V, W], t: Set[W]) {
    is_independent_set(f.dst, t) implies
        is_independent_set(f.src, simple_graph_hom_preimage_set(f, t))
} by {
    if is_independent_set(f.dst, t) {
        is_independent_set(f.dst, t) = forall(y1: W, y2: W) {
            t.contains(y1) and t.contains(y2) and y1 != y2 implies not f.dst.adj(y1, y2)
        }
        forall(x1: V, x2: V) {
            if simple_graph_hom_preimage_set(f, t).contains(x1)
                and simple_graph_hom_preimage_set(f, t).contains(x2) and x1 != x2 {
                simple_graph_hom_preimage_set_contains_eq(f, t, x1)
                simple_graph_hom_preimage_set_contains_eq(f, t, x2)
                t.contains(f.map(x1))
                t.contains(f.map(x2))
                if f.src.adj(x1, x2) {
                    simple_graph_hom_maps_adj(f, x1, x2)
                    f.dst.adj(f.map(x1), f.map(x2))
                    simple_graph_adj_ne(f.dst, f.map(x1), f.map(x2))
                    f.map(x1) != f.map(x2)
                    is_independent_set(f.dst, t) and t.contains(f.map(x1)) and
                        t.contains(f.map(x2)) and f.map(x1) != f.map(x2)
                    is_independent_set_contains_not_adj(f.dst, t, f.map(x1), f.map(x2))
                    not f.dst.adj(f.map(x1), f.map(x2))
                    false
                }
                not f.src.adj(x1, x2)
            }
        }
        is_independent_set(f.src, simple_graph_hom_preimage_set(f, t))
    }
}
