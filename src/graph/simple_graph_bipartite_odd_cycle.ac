from nat import Nat, pos_of_ne_zero
from list import List, reverse, reverse_length, add_length
from finite_set import FiniteSet
from graph.simple_graph import SimpleGraph, simple_graph_adj_symmetric
from graph.simple_graph_bipartite import is_bipartition, is_bipartition_apply,
    is_bipartite, is_bipartite_intro, is_bipartite_witness, is_bipartition_intro
from graph.simple_graph_walks import simple_graph_walk, simple_graph_walk_nil,
    simple_graph_walk_cons_iff, simple_graph_closed_walk, simple_graph_closed_walk_is_walk,
    simple_graph_walk_append_adj, simple_graph_walk_of_adj, simple_graph_closed_walk_intro
from graph.simple_graph_tree import simple_graph_cycle
from data.list.list_cons_membership import cons_contains_head, cons_contains_of_tail_contains,
    nil_not_contains
from data.list.list_reverse_append_helpers import reverse_cons_eq_append, reverse_nil

numerals Nat

/// True when the natural number `n` is odd.
define nat_odd(n: Nat) -> Bool {
    match n {
        Nat.zero {
            false
        }
        Nat.suc(pred) {
            not nat_odd(pred)
        }
    }
}

/// Zero is even.
theorem nat_odd_zero {
    nat_odd(Nat.0) = false
}

/// Oddness flips under successor.
theorem nat_odd_suc(n: Nat) {
    nat_odd(n.suc) = not nat_odd(n)
}

/// Distinct booleans flip equality with a third boolean: if `x != y`, then `x = z` holds
/// exactly when `y != z`.
theorem bool_ne_eq_flip(x: Bool, y: Bool, z: Bool) {
    x != y implies (x = z) = not (y = z)
} by {
    if x != y {
        if x {
            (x = z) = not (y = z)
        } else {
            (x = z) = not (y = z)
        }
    }
}

/// True when every target of the walk `steps` lies in `s`.
///
/// The recursion makes the empty walk trivially in `s` by reduction, and a prepended head is
/// in `s` exactly when it is a member and the tail is.
define walk_in_set[V](s: FiniteSet[V], steps: List[V]) -> Bool {
    match steps {
        List.nil {
            true
        }
        List.cons(head, tail) {
            s.contains(head) and walk_in_set(s, tail)
        }
    }
}

/// A prepended walk stays in `s` only when its head is a member of `s`.
theorem walk_in_set_cons_head[V](s: FiniteSet[V], h: V, t: List[V]) {
    walk_in_set(s, List.cons(h, t)) implies s.contains(h)
} by {
    if walk_in_set(s, List.cons(h, t)) {
        s.contains(h)
    }
}

/// A prepended walk stays in `s` only when its tail does.
theorem walk_in_set_cons_tail[V](s: FiniteSet[V], h: V, t: List[V]) {
    walk_in_set(s, List.cons(h, t)) implies walk_in_set(s, t)
} by {
    if walk_in_set(s, List.cons(h, t)) {
        walk_in_set(s, t)
    }
}

/// A prepended target is in `s` exactly when the head is a member and the tail stays in `s`.
theorem walk_in_set_cons[V](s: FiniteSet[V], h: V, t: List[V]) {
    walk_in_set(s, List.cons(h, t)) = (s.contains(h) and walk_in_set(s, t))
} by {
    if walk_in_set(s, List.cons(h, t)) {
        walk_in_set_cons_head(s, h, t)
        s.contains(h)
        walk_in_set_cons_tail(s, h, t)
        walk_in_set(s, t)
        s.contains(h) and walk_in_set(s, t)
    }
    if s.contains(h) and walk_in_set(s, t) {
        walk_in_set(s, List.cons(h, t))
    }
}

/// The empty walk stays inside every set.
theorem walk_in_set_nil[V](s: FiniteSet[V]) {
    walk_in_set(s, List.nil[V])
}

/// A one-target walk stays inside `s` when its target does.
theorem walk_in_set_singleton[V](s: FiniteSet[V], y: V) {
    s.contains(y) implies walk_in_set(s, List.singleton(y))
} by {
    if s.contains(y) {
        List.singleton(y) = List.cons(y, List.nil[V])
        walk_in_set(s, List.singleton(y)) = (s.contains(y) and walk_in_set(s, List.nil[V]))
        walk_in_set(s, List.nil[V])
        s.contains(y) and walk_in_set(s, List.nil[V])
        walk_in_set(s, List.singleton(y))
    }
}

/// A boolean differs from `false` exactly when it holds.
theorem bool_ne_false(p: Bool) {
    (p != false) = p
} by {
    if p {
        (p != false) = p
    } else {
        (p != false) = p
    }
}

/// Negating a boolean inequality flips the second operand's negation.
theorem bool_ne_not(p: Bool, q: Bool) {
    not (p != q) = (p != not q)
} by {
    if p {
        if q {
            not (p != q) = (p != not q)
        } else {
            not (p != q) = (p != not q)
        }
    } else {
        if q {
            not (p != q) = (p != not q)
        } else {
            not (p != q) = (p != not q)
        }
    }
}

/// A sum of naturals is odd exactly when its two summands have different parities.
theorem nat_odd_add(a: Nat, b: Nat) {
    nat_odd(a + b) = (nat_odd(a) != nat_odd(b))
} by {
    define p(k: Nat) -> Bool {
        nat_odd(a + k) = (nat_odd(a) != nat_odd(k))
    }

    nat_odd(a + Nat.0) = nat_odd(a)
    nat_odd(Nat.0) = false
    bool_ne_false(nat_odd(a))
    (nat_odd(a) != false) = nat_odd(a)
    nat_odd(a) = (nat_odd(a) != nat_odd(Nat.0))
    nat_odd(a + Nat.0) = (nat_odd(a) != nat_odd(Nat.0))
    p(Nat.0)

    forall(k: Nat) {
        if p(k) {
            nat_odd(a + k.suc) = not nat_odd(a + k)
            nat_odd(k.suc) = not nat_odd(k)
            p(k)
            nat_odd(a + k) = (nat_odd(a) != nat_odd(k))
            not nat_odd(a + k) = not (nat_odd(a) != nat_odd(k))
            nat_odd(a + k.suc) = not (nat_odd(a) != nat_odd(k))
            bool_ne_not(nat_odd(a), nat_odd(k))
            not (nat_odd(a) != nat_odd(k)) = (nat_odd(a) != not nat_odd(k))
            nat_odd(a + k.suc) = (nat_odd(a) != not nat_odd(k))
            (nat_odd(a) != not nat_odd(k)) = (nat_odd(a) != nat_odd(k.suc))
            nat_odd(a + k.suc) = (nat_odd(a) != nat_odd(k.suc))
            p(k.suc)
        }
    }
    p(Nat.0) and forall(k: Nat) { p(k) implies p(k.suc) }
    Nat.induction(p)
    p(b)
}

/// Concatenating the target lists of two walks gives a walk between the outer endpoints.
theorem simple_graph_walk_concat[V](
    g: SimpleGraph[V], a: V, m: V, b: V, w1: List[V], w2: List[V]
) {
    simple_graph_walk(g, a, m, w1) and simple_graph_walk(g, m, b, w2)
        implies simple_graph_walk(g, a, b, w1 + w2)
} by {
    define pc(xs: List[V], u: V, v: V, w: V) -> Bool {
        simple_graph_walk(g, u, v, xs) and simple_graph_walk(g, v, w, w2)
            implies simple_graph_walk(g, u, w, xs + w2)
    }

    define p(xs: List[V]) -> Bool {
        forall(u: V, v: V, w: V) {
            pc(xs, u, v, w)
        }
    }

    forall(u: V, v: V, w: V) {
        if simple_graph_walk(g, u, v, List.nil[V]) and simple_graph_walk(g, v, w, w2) {
            simple_graph_walk_nil(g, u, v)
            u = v
            List.nil[V] + w2 = w2
            simple_graph_walk(g, u, w, List.nil[V] + w2)
            pc(List.nil[V], u, v, w)
        }
        pc(List.nil[V], u, v, w)
    }
    p(List.nil[V])

    forall(h: V, t: List[V]) {
        if p(t) {
            forall(u: V, v: V, w: V) {
                if simple_graph_walk(g, u, v, List.cons(h, t)) and simple_graph_walk(g, v, w, w2) {
                    simple_graph_walk_cons_iff(g, u, v, h, t)
                    g.adj(u, h)
                    simple_graph_walk(g, h, v, t)
                    p(t)
                    pc(t, h, v, w)
                    pc(t, h, v, w) = (simple_graph_walk(g, h, v, t) and simple_graph_walk(g, v, w, w2) implies simple_graph_walk(g, h, w, t + w2))
                    simple_graph_walk(g, h, w, t + w2)
                    List.cons(h, t) + w2 = List.cons(h, t + w2)
                    simple_graph_walk_cons_iff(g, u, w, h, t + w2)
                    g.adj(u, h) and simple_graph_walk(g, h, w, t + w2) implies simple_graph_walk(g, u, w, List.cons(h, t + w2))
                    simple_graph_walk(g, u, w, List.cons(h, t + w2))
                    simple_graph_walk(g, u, w, List.cons(h, t) + w2)
                    pc(List.cons(h, t), u, v, w)
                }
                pc(List.cons(h, t), u, v, w)
            }
            p(List.cons(h, t))
        }
    }
    forall(h: V, t: List[V]) { p(t) implies p(List.cons(h, t)) }
    function[T0](x0: List[T0] -> Bool) {
        not forall(x1: T0, x2: List[T0]) { not x0(x2) or x0(List.cons(x1, x2)) } or
        not x0(List.nil[T0]) or forall(x3: List[T0]) { x0(x3) } = true
    }[V](p)
    function[T0](x0: List[T0] -> Bool, x1: List[T0]) {
        not forall(x2: T0, x3: List[T0]) { not x0(x3) or x0(List.cons(x2, x3)) } or
        not x0(List.nil[T0]) or x0(x1)
    }[V](p, w1)
    p(w1)
    pc(w1, a, m, b)
    pc(w1, a, m, b) = (simple_graph_walk(g, a, m, w1) and simple_graph_walk(g, m, b, w2) implies simple_graph_walk(g, a, b, w1 + w2))
    if simple_graph_walk(g, a, m, w1) and simple_graph_walk(g, m, b, w2) {
        simple_graph_walk(g, a, b, w1 + w2)
    }
}

/// A concatenated walk splits at the concatenation point.
theorem simple_graph_walk_concat_split[V](g: SimpleGraph[V], a: V, b: V, w1: List[V], w2: List[V]) {
    simple_graph_walk(g, a, b, w1 + w2) implies exists(m: V) {
        simple_graph_walk(g, a, m, w1) and simple_graph_walk(g, m, b, w2)
    }
} by {
    define pc(xs: List[V], u: V, v: V) -> Bool {
        simple_graph_walk(g, u, v, xs + w2) implies exists(m: V) {
            simple_graph_walk(g, u, m, xs) and simple_graph_walk(g, m, v, w2)
        }
    }

    define p(xs: List[V]) -> Bool {
        forall(u: V, v: V) {
            pc(xs, u, v)
        }
    }

    forall(u: V, v: V) {
        if simple_graph_walk(g, u, v, List.nil[V] + w2) {
            List.nil[V] + w2 = w2
            simple_graph_walk(g, u, v, w2)
            exists(m: V) {
                simple_graph_walk(g, u, m, List.nil[V]) and simple_graph_walk(g, m, v, w2)
            }
            pc(List.nil[V], u, v)
        }
        pc(List.nil[V], u, v)
    }
    p(List.nil[V])

    forall(h: V, t: List[V]) {
        if p(t) {
            forall(u: V, v: V) {
                if simple_graph_walk(g, u, v, List.cons(h, t) + w2) {
                    List.cons(h, t) + w2 = List.cons(h, t + w2)
                    simple_graph_walk(g, u, v, List.cons(h, t + w2))
                    simple_graph_walk_cons_iff(g, u, v, h, t + w2)
                    g.adj(u, h)
                    simple_graph_walk(g, h, v, t + w2)
                    p(t)
                    pc(t, h, v)
                    pc(t, h, v) = (simple_graph_walk(g, h, v, t + w2) implies exists(m: V) {
                        simple_graph_walk(g, h, m, t) and simple_graph_walk(g, m, v, w2)
                    })
                    exists(m: V) {
                        simple_graph_walk(g, h, m, t) and simple_graph_walk(g, m, v, w2)
                    }
                    let (m: V) satisfy {
                        simple_graph_walk(g, h, m, t) and simple_graph_walk(g, m, v, w2)
                    }
                    simple_graph_walk(g, u, m, List.cons(h, t))
                    exists(m2: V) {
                        simple_graph_walk(g, u, m2, List.cons(h, t)) and simple_graph_walk(g, m2, v, w2)
                    }
                    pc(List.cons(h, t), u, v)
                }
                pc(List.cons(h, t), u, v)
            }
            p(List.cons(h, t))
        }
    }
    forall(h: V, t: List[V]) { p(t) implies p(List.cons(h, t)) }
    function[T0](x0: List[T0] -> Bool) {
        not forall(x1: T0, x2: List[T0]) { not x0(x2) or x0(List.cons(x1, x2)) } or
        not x0(List.nil[T0]) or forall(x3: List[T0]) { x0(x3) } = true
    }[V](p)
    function[T0](x0: List[T0] -> Bool, x1: List[T0]) {
        not forall(x2: T0, x3: List[T0]) { not x0(x3) or x0(List.cons(x2, x3)) } or
        not x0(List.nil[T0]) or x0(x1)
    }[V](p, w1)
    p(w1)
    pc(w1, a, b)
    pc(w1, a, b) = (simple_graph_walk(g, a, b, w1 + w2) implies exists(m: V) {
        simple_graph_walk(g, a, m, w1) and simple_graph_walk(g, m, b, w2)
    })
    if simple_graph_walk(g, a, b, w1 + w2) {
        exists(m: V) {
            simple_graph_walk(g, a, m, w1) and simple_graph_walk(g, m, b, w2)
        }
    }
}

/// A walk whose targets lie in `s` keeps that property when extended on the left.
theorem walk_in_set_left[V](s: FiniteSet[V], w1: List[V], w2: List[V]) {
    walk_in_set(s, w1 + w2) implies walk_in_set(s, w1)
} by {
    define pc(xs: List[V]) -> Bool {
        walk_in_set(s, xs + w2) implies walk_in_set(s, xs)
    }

    pc(List.nil[V])

    forall(h: V, t: List[V]) {
        if pc(t) {
            if walk_in_set(s, List.cons(h, t) + w2) {
                walk_in_set(s, List.cons(h, t) + w2)
                List.cons(h, t) + w2 = List.cons(h, t + w2)
                walk_in_set(s, List.cons(h, t + w2))
                walk_in_set(s, List.cons(h, t + w2)) = (s.contains(h) and walk_in_set(s, t + w2))
                s.contains(h) and walk_in_set(s, t + w2)
                s.contains(h)
                walk_in_set(s, t + w2)
                pc(t)
                walk_in_set(s, t)
                walk_in_set(s, List.cons(h, t)) = (s.contains(h) and walk_in_set(s, t))
                s.contains(h) and walk_in_set(s, t)
                walk_in_set(s, List.cons(h, t))
                pc(List.cons(h, t))
            }
            pc(List.cons(h, t))
        }
    }
    forall(h: V, t: List[V]) { pc(t) implies pc(List.cons(h, t)) }
    function[T0](x0: List[T0] -> Bool) {
        not forall(x1: T0, x2: List[T0]) { not x0(x2) or x0(List.cons(x1, x2)) } or
        not x0(List.nil[T0]) or forall(x3: List[T0]) { x0(x3) } = true
    }[V](pc)
    function[T0](x0: List[T0] -> Bool, x1: List[T0]) {
        not forall(x2: T0, x3: List[T0]) { not x0(x3) or x0(List.cons(x2, x3)) } or
        not x0(List.nil[T0]) or x0(x1)
    }[V](pc, w1)
    pc(w1)
}

/// A walk whose targets lie in `s` keeps that property when extended on the right.
theorem walk_in_set_right[V](s: FiniteSet[V], w1: List[V], w2: List[V]) {
    walk_in_set(s, w1 + w2) implies walk_in_set(s, w2)
} by {
    define pc(xs: List[V]) -> Bool {
        walk_in_set(s, xs + w2) implies walk_in_set(s, w2)
    }

    List.nil[V] + w2 = w2
    walk_in_set(s, List.nil[V] + w2) implies walk_in_set(s, w2)
    pc(List.nil[V])

    forall(h: V, t: List[V]) {
        if pc(t) {
            if walk_in_set(s, List.cons(h, t) + w2) {
                List.cons(h, t) + w2 = List.cons(h, t + w2)
                walk_in_set(s, List.cons(h, t + w2)) = (s.contains(h) and walk_in_set(s, t + w2))
                walk_in_set(s, t + w2)
                pc(t)
                walk_in_set(s, w2)
                pc(List.cons(h, t))
            }
            pc(List.cons(h, t))
        }
    }
    forall(h: V, t: List[V]) { pc(t) implies pc(List.cons(h, t)) }
    function[T0](x0: List[T0] -> Bool) {
        not forall(x1: T0, x2: List[T0]) { not x0(x2) or x0(List.cons(x1, x2)) } or
        not x0(List.nil[T0]) or forall(x3: List[T0]) { x0(x3) } = true
    }[V](pc)
    function[T0](x0: List[T0] -> Bool, x1: List[T0]) {
        not forall(x2: T0, x3: List[T0]) { not x0(x3) or x0(List.cons(x2, x3)) } or
        not x0(List.nil[T0]) or x0(x1)
    }[V](pc, w1)
    pc(w1)
}

/// Two walks that stay in `s` concatenate to a walk that stays in `s`.
theorem walk_in_set_concat[V](s: FiniteSet[V], w1: List[V], w2: List[V]) {
    walk_in_set(s, w1) and walk_in_set(s, w2) implies walk_in_set(s, w1 + w2)
} by {
    define pc(xs: List[V]) -> Bool {
        walk_in_set(s, xs) and walk_in_set(s, w2) implies walk_in_set(s, xs + w2)
    }

    List.nil[V] + w2 = w2
    pc(List.nil[V])

    forall(h: V, t: List[V]) {
        if pc(t) {
            if walk_in_set(s, List.cons(h, t)) and walk_in_set(s, w2) {
                walk_in_set(s, List.cons(h, t)) = (s.contains(h) and walk_in_set(s, t))
                s.contains(h) and walk_in_set(s, t)
                s.contains(h)
                walk_in_set(s, t)
                pc(t)
                walk_in_set(s, t + w2)
                List.cons(h, t) + w2 = List.cons(h, t + w2)
                walk_in_set(s, List.cons(h, t + w2)) = (s.contains(h) and walk_in_set(s, t + w2))
                s.contains(h) and walk_in_set(s, t + w2)
                walk_in_set(s, List.cons(h, t) + w2)
                pc(List.cons(h, t))
            }
            pc(List.cons(h, t))
        }
    }
    forall(h: V, t: List[V]) { pc(t) implies pc(List.cons(h, t)) }
    function[T0](x0: List[T0] -> Bool) {
        not forall(x1: T0, x2: List[T0]) { not x0(x2) or x0(List.cons(x1, x2)) } or
        not x0(List.nil[T0]) or forall(x3: List[T0]) { x0(x3) } = true
    }[V](pc)
    function[T0](x0: List[T0] -> Bool, x1: List[T0]) {
        not forall(x2: T0, x3: List[T0]) { not x0(x3) or x0(List.cons(x2, x3)) } or
        not x0(List.nil[T0]) or x0(x1)
    }[V](pc, w1)
    pc(w1)
}

/// The colour of a walk endpoint flips once more when one step is prepended: distinct sides
/// at `a` and `next` and an even tail keep the endpoints of the extended walk on the same
/// side exactly when the extended length is even.
theorem bool_ne_eq_trans[V](part: V -> Bool, a: V, b: V, next: V, tail: List[V]) {
    part(a) != part(next) and (part(next) = part(b)) = not nat_odd(tail.length) and
        nat_odd(List.cons(next, tail).length) = not nat_odd(tail.length)
        implies (part(a) = part(b)) = not nat_odd(List.cons(next, tail).length)
} by {
    if part(a) != part(next) and (part(next) = part(b)) = not nat_odd(tail.length) and
        nat_odd(List.cons(next, tail).length) = not nat_odd(tail.length) {
        bool_ne_eq_flip(part(a), part(next), part(b))
        part(a) != part(next) implies (part(a) = part(b)) = not (part(next) = part(b))
        (part(a) = part(b)) = not (part(next) = part(b))
        (part(next) = part(b)) = not nat_odd(tail.length)
        nat_odd(List.cons(next, tail).length) = not nat_odd(tail.length)
        (part(a) = part(b)) = not nat_odd(List.cons(next, tail).length)
    }
}

/// A walk in a bipartite graph flips the side once per edge, so its endpoints lie on the
/// same side exactly when its length is even.
theorem bipartition_walk_color_flip[V](
    g: SimpleGraph[V], s: FiniteSet[V], part: V -> Bool, x: V, y: V, steps: List[V]
) {
    is_bipartition(g, s, part) and simple_graph_walk(g, x, y, steps) and s.contains(x) and
        walk_in_set(s, steps)
        implies (part(x) = part(y)) = not nat_odd(steps.length)
} by {
    define p(xs: List[V]) -> Bool {
        forall(a: V, b: V) {
            is_bipartition(g, s, part) and simple_graph_walk(g, a, b, xs) and s.contains(a) and
                walk_in_set(s, xs)
                implies (part(a) = part(b)) = not nat_odd(xs.length)
        }
    }

    forall(a: V, b: V) {
        if is_bipartition(g, s, part) and simple_graph_walk(g, a, b, List.nil[V]) and s.contains(a) and
            walk_in_set(s, List.nil[V]) {
            simple_graph_walk_nil(g, a, b)
            a = b
            (part(a) = part(b)) = not nat_odd(List.nil[V].length)
        }
        is_bipartition(g, s, part) and simple_graph_walk(g, a, b, List.nil[V]) and s.contains(a) and
            walk_in_set(s, List.nil[V]) implies (part(a) = part(b)) = not nat_odd(List.nil[V].length)
    }
    p(List.nil[V])

    forall(next: V, tail: List[V]) {
        if p(tail) {
            forall(a: V, b: V) {
                if is_bipartition(g, s, part) and simple_graph_walk(g, a, b, List.cons(next, tail)) and
                    s.contains(a) and walk_in_set(s, List.cons(next, tail)) {
                    simple_graph_walk_cons_iff(g, a, b, next, tail)
                    g.adj(a, next)
                    simple_graph_walk(g, next, b, tail)
                    walk_in_set(s, List.cons(next, tail)) = (s.contains(next) and walk_in_set(s, tail))
                    s.contains(next) and walk_in_set(s, tail)
                    s.contains(next)
                    walk_in_set(s, tail)
                    is_bipartition_apply(g, s, part, a, next)
                    part(a) != part(next)
                    p(tail)
                    p(tail) = forall(u: V, w: V) {
                        is_bipartition(g, s, part) and simple_graph_walk(g, u, w, tail) and s.contains(u) and walk_in_set(s, tail) implies (part(u) = part(w)) = not nat_odd(tail.length)
                    }
                    forall(u: V, w: V) {
                        is_bipartition(g, s, part) and simple_graph_walk(g, u, w, tail) and s.contains(u) and walk_in_set(s, tail) implies (part(u) = part(w)) = not nat_odd(tail.length)
                    }
                    is_bipartition(g, s, part) and simple_graph_walk(g, next, b, tail) and s.contains(next) and walk_in_set(s, tail) implies (part(next) = part(b)) = not nat_odd(tail.length)
                    (part(next) = part(b)) = not nat_odd(tail.length)
                    bool_ne_eq_flip(part(a), part(next), part(b))
                    part(a) != part(next) implies (part(a) = part(b)) = not (part(next) = part(b))
                    (part(a) = part(b)) = not (part(next) = part(b))
                    (part(next) = part(b)) = not nat_odd(tail.length)
                    List.cons(next, tail).length = tail.length.suc
                    nat_odd_suc(tail.length)
                    nat_odd(tail.length.suc) = not nat_odd(tail.length)
                    nat_odd(List.cons(next, tail).length) = not nat_odd(tail.length)
                    bool_ne_eq_trans(part, a, b, next, tail)
                    (part(a) = part(b)) = not nat_odd(List.cons(next, tail).length)
                }
                is_bipartition(g, s, part) and simple_graph_walk(g, a, b, List.cons(next, tail)) and s.contains(a) and walk_in_set(s, List.cons(next, tail)) implies (part(a) = part(b)) = not nat_odd(List.cons(next, tail).length)
            }
            p(List.cons(next, tail))
        }
    }
    forall(next: V, tail: List[V]) { p(tail) implies p(List.cons(next, tail)) }
    function[T0](x0: List[T0] -> Bool) {
        not forall(x1: T0, x2: List[T0]) { not x0(x2) or x0(List.cons(x1, x2)) } or
        not x0(List.nil[T0]) or forall(x3: List[T0]) { x0(x3) } = true
    }[V](p)
    function[T0](x0: List[T0] -> Bool, x1: List[T0]) {
        not forall(x2: T0, x3: List[T0]) { not x0(x3) or x0(List.cons(x2, x3)) } or
        not x0(List.nil[T0]) or x0(x1)
    }[V](p, steps)
    p(steps)
    if is_bipartition(g, s, part) and simple_graph_walk(g, x, y, steps) and s.contains(x) and
        walk_in_set(s, steps) {
        (part(x) = part(y)) = not nat_odd(steps.length)
    }
}

/// A closed walk of a bipartite graph has even length.
theorem is_bipartition_no_odd_closed_walk[V](
    g: SimpleGraph[V], s: FiniteSet[V], part: V -> Bool, x: V, steps: List[V]
) {
    is_bipartition(g, s, part) and simple_graph_closed_walk(g, x, steps) and s.contains(x) and
        walk_in_set(s, steps)
        implies not nat_odd(steps.length)
} by {
    if is_bipartition(g, s, part) and simple_graph_closed_walk(g, x, steps) and s.contains(x) and
        walk_in_set(s, steps) {
        simple_graph_closed_walk_is_walk(g, x, steps)
        simple_graph_walk(g, x, x, steps)
        bipartition_walk_color_flip(g, s, part, x, x, steps)
        (part(x) = part(x)) = not nat_odd(steps.length)
        not nat_odd(steps.length)
    }
}

/// True when `g` contains an odd cycle whose vertices all lie in `s`.
define has_odd_cycle[V](g: SimpleGraph[V], s: FiniteSet[V]) -> Bool {
    exists(start: V, steps: List[V]) {
        s.contains(start) and simple_graph_cycle(g, start, steps) and nat_odd(steps.length) and
            walk_in_set(s, steps)
    }
}

/// A bipartite graph has no odd cycle: every closed walk alternates between the two sides,
/// so its length is even.
theorem is_bipartite_no_odd_cycle[V](g: SimpleGraph[V], s: FiniteSet[V]) {
    is_bipartite(g, s) implies not has_odd_cycle(g, s)
} by {
    if is_bipartite(g, s) {
        is_bipartite_witness(g, s)
        let (part: V -> Bool) satisfy {
            is_bipartition(g, s, part)
        }
        if has_odd_cycle(g, s) {
            has_odd_cycle(g, s) = exists(start: V, steps: List[V]) {
                s.contains(start) and simple_graph_cycle(g, start, steps) and nat_odd(steps.length) and
                    walk_in_set(s, steps)
            }
            exists(start: V, steps: List[V]) {
                s.contains(start) and simple_graph_cycle(g, start, steps) and nat_odd(steps.length) and
                    walk_in_set(s, steps)
            }
            let (start: V, steps: List[V]) satisfy {
                s.contains(start) and simple_graph_cycle(g, start, steps) and nat_odd(steps.length) and
                    walk_in_set(s, steps)
            }
            simple_graph_cycle(g, start, steps) = (
                simple_graph_closed_walk(g, start, steps) and Nat.3 <= steps.length and steps.is_unique
            )
            simple_graph_closed_walk(g, start, steps) and Nat.3 <= steps.length and steps.is_unique
            simple_graph_closed_walk(g, start, steps)
            is_bipartition_no_odd_closed_walk(g, s, part, start, steps)
            not nat_odd(steps.length)
            false
        }
        not has_odd_cycle(g, s)
    }
}

/// The target list of the reversed walk: a walk `start -> ... -> finish` with target list
/// `steps` becomes a walk `finish -> ... -> start` with targets `rev_walk_targets(steps, start)`.
define rev_walk_targets[V](steps: List[V], start: V) -> List[V] {
    match steps {
        List.nil {
            List.nil[V]
        }
        List.cons(head, tail) {
            rev_walk_targets(tail, head) + List.cons(start, List.nil[V])
        }
    }
}

/// Reversing a walk gives a walk back to the start.
theorem simple_graph_walk_reverse[V](g: SimpleGraph[V], a: V, b: V, steps: List[V]) {
    simple_graph_walk(g, a, b, steps) implies simple_graph_walk(g, b, a, rev_walk_targets(steps, a))
} by {
    define pc(xs: List[V], u: V, v: V) -> Bool {
        simple_graph_walk(g, u, v, xs) implies simple_graph_walk(g, v, u, rev_walk_targets(xs, u))
    }

    define p(xs: List[V]) -> Bool {
        forall(u: V, v: V) {
            pc(xs, u, v)
        }
    }

    forall(u: V, v: V) {
        if simple_graph_walk(g, u, v, List.nil[V]) {
            simple_graph_walk_nil(g, u, v)
            u = v
            rev_walk_targets(List.nil[V], u) = List.nil[V]
            simple_graph_walk(g, v, u, rev_walk_targets(List.nil[V], u))
            pc(List.nil[V], u, v)
        }
        pc(List.nil[V], u, v)
    }
    p(List.nil[V])

    forall(h: V, t: List[V]) {
        if p(t) {
            forall(u: V, v: V) {
                if simple_graph_walk(g, u, v, List.cons(h, t)) {
                    simple_graph_walk_cons_iff(g, u, v, h, t)
                    g.adj(u, h)
                    simple_graph_walk(g, h, v, t)
                    p(t)
                    pc(t, h, v)
                    pc(t, h, v) = (simple_graph_walk(g, h, v, t) implies simple_graph_walk(g, v, h, rev_walk_targets(t, h)))
                    simple_graph_walk(g, v, h, rev_walk_targets(t, h))
                    rev_walk_targets(List.cons(h, t), u) = rev_walk_targets(t, h) + List.cons(u, List.nil[V])
                    simple_graph_adj_symmetric(g, u, h)
                    g.adj(h, u)
                    simple_graph_walk_of_adj(g, h, u)
                    simple_graph_walk(g, h, u, List.singleton(u))
                    simple_graph_walk_concat(g, v, h, u, rev_walk_targets(t, h), List.singleton(u))
                    simple_graph_walk(g, v, u, rev_walk_targets(t, h) + List.singleton(u))
                    simple_graph_walk(g, v, u, rev_walk_targets(List.cons(h, t), u))
                    pc(List.cons(h, t), u, v)
                }
                pc(List.cons(h, t), u, v)
            }
            p(List.cons(h, t))
        }
    }
    forall(h: V, t: List[V]) { p(t) implies p(List.cons(h, t)) }
    function[T0](x0: List[T0] -> Bool) {
        not forall(x1: T0, x2: List[T0]) { not x0(x2) or x0(List.cons(x1, x2)) } or
        not x0(List.nil[T0]) or forall(x3: List[T0]) { x0(x3) } = true
    }[V](p)
    function[T0](x0: List[T0] -> Bool, x1: List[T0]) {
        not forall(x2: T0, x3: List[T0]) { not x0(x3) or x0(List.cons(x2, x3)) } or
        not x0(List.nil[T0]) or x0(x1)
    }[V](p, steps)
    p(steps)
    pc(steps, a, b)
    if simple_graph_walk(g, a, b, steps) {
        simple_graph_walk(g, b, a, rev_walk_targets(steps, a))
    }
}

/// The reversed walk's targets lie in `s` whenever the original walk's do and the start does.
theorem rev_walk_targets_in_set[V](s: FiniteSet[V], steps: List[V], start: V) {
    walk_in_set(s, steps) and s.contains(start) implies walk_in_set(s, rev_walk_targets(steps, start))
} by {
    define pc(xs: List[V]) -> Bool {
        forall(z: V) {
            walk_in_set(s, xs) and s.contains(z) implies walk_in_set(s, rev_walk_targets(xs, z))
        }
    }

    forall(z: V) {
        if walk_in_set(s, List.nil[V]) and s.contains(z) {
            walk_in_set(s, List.nil[V])
        }
        walk_in_set(s, List.nil[V]) and s.contains(z) implies walk_in_set(s, rev_walk_targets(List.nil[V], z))
    }
    pc(List.nil[V])

    forall(h: V, t: List[V]) {
        if pc(t) {
            forall(z: V) {
                if walk_in_set(s, List.cons(h, t)) and s.contains(z) {
                    walk_in_set(s, List.cons(h, t)) = (s.contains(h) and walk_in_set(s, t))
                    s.contains(h) and walk_in_set(s, t)
                    s.contains(h)
                    walk_in_set(s, t)
                    pc(t)
                    walk_in_set(s, t) and s.contains(h) implies walk_in_set(s, rev_walk_targets(t, h))
                    walk_in_set(s, rev_walk_targets(t, h))
                    rev_walk_targets(List.cons(h, t), z) = rev_walk_targets(t, h) + List.cons(z, List.nil[V])
                    walk_in_set_singleton(s, z)
                    walk_in_set(s, List.singleton(z))
                    walk_in_set_concat(s, rev_walk_targets(t, h), List.singleton(z))
                    walk_in_set(s, rev_walk_targets(t, h) + List.singleton(z))
                    walk_in_set(s, rev_walk_targets(List.cons(h, t), z))
                }
                walk_in_set(s, List.cons(h, t)) and s.contains(z) implies walk_in_set(s, rev_walk_targets(List.cons(h, t), z))
            }
            pc(List.cons(h, t))
        }
    }
    forall(h: V, t: List[V]) { pc(t) implies pc(List.cons(h, t)) }
    function[T0](x0: List[T0] -> Bool) {
        not forall(x1: T0, x2: List[T0]) { not x0(x2) or x0(List.cons(x1, x2)) } or
        not x0(List.nil[T0]) or forall(x3: List[T0]) { x0(x3) } = true
    }[V](pc)
    function[T0](x0: List[T0] -> Bool, x1: List[T0]) {
        not forall(x2: T0, x3: List[T0]) { not x0(x3) or x0(List.cons(x2, x3)) } or
        not x0(List.nil[T0]) or x0(x1)
    }[V](pc, steps)
    pc(steps)
    if walk_in_set(s, steps) and s.contains(start) {
        walk_in_set(s, rev_walk_targets(steps, start))
    }
}

/// An odd natural number is nonzero.
theorem nat_odd_ne_zero(n: Nat) {
    nat_odd(n) implies n != Nat.0
} by {
    if nat_odd(n) {
        if n = Nat.0 {
            nat_odd_zero
            nat_odd(n) = false
            false
        }
        n != Nat.0
    }
}

/// Distinct truth values: `false` and `true` differ.
theorem bool_ne_of_not_and(a: Bool, b: Bool) {
    not a and b implies a != b
} by {
    if not a and b {
        if a = b {
            b
            a
            false
        }
        a != b
    }
}

/// True when `g` contains an odd closed walk whose targets all lie in `s`.
define has_odd_closed_walk[V](g: SimpleGraph[V], s: FiniteSet[V]) -> Bool {
    exists(x: V, steps: List[V]) {
        simple_graph_closed_walk(g, x, steps) and nat_odd(steps.length) and
            walk_in_set(s, steps)
    }
}

/// The reversed-walk target list has the same length as the original walk.
theorem rev_walk_targets_length[V](steps: List[V], start: V) {
    rev_walk_targets(steps, start).length = steps.length
} by {
    define pc(xs: List[V]) -> Bool {
        forall(z: V) {
            rev_walk_targets(xs, z).length = xs.length
        }
    }

    forall(z: V) {
        rev_walk_targets(List.nil[V], z) = List.nil[V]
        rev_walk_targets(List.nil[V], z).length = List.nil[V].length
    }
    pc(List.nil[V])

    forall(h: V, t: List[V]) {
        if pc(t) {
            forall(z: V) {
                rev_walk_targets(List.cons(h, t), z) = rev_walk_targets(t, h) + List.cons(z, List.nil[V])
                add_length(rev_walk_targets(t, h), List.cons(z, List.nil[V]))
                rev_walk_targets(t, h).length + List.cons(z, List.nil[V]).length = (rev_walk_targets(t, h) + List.cons(z, List.nil[V])).length
                pc(t)
                rev_walk_targets(t, h).length = t.length
                List.cons(z, List.nil[V]).length = List.nil[V].length.suc
                rev_walk_targets(List.cons(h, t), z).length = t.length + List.nil[V].length.suc
                List.cons(h, t).length = t.length.suc
                t.length + List.nil[V].length.suc = t.length.suc
                rev_walk_targets(List.cons(h, t), z).length = List.cons(h, t).length
            }
            pc(List.cons(h, t))
        }
    }
    forall(h: V, t: List[V]) { pc(t) implies pc(List.cons(h, t)) }
    function[T0](x0: List[T0] -> Bool) {
        not forall(x1: T0, x2: List[T0]) { not x0(x2) or x0(List.cons(x1, x2)) } or
        not x0(List.nil[T0]) or forall(x3: List[T0]) { x0(x3) } = true
    }[V](pc)
    function[T0](x0: List[T0] -> Bool, x1: List[T0]) {
        not forall(x2: T0, x3: List[T0]) { not x0(x3) or x0(List.cons(x2, x3)) } or
        not x0(List.nil[T0]) or x0(x1)
    }[V](pc, steps)
    pc(steps)
}

/// Distinct truth values: `true` and `false` differ.
theorem bool_ne_of_and_not(a: Bool, b: Bool) {
    a and not b implies a != b
} by {
    if a and not b {
        if a = b {
            b
            a
            false
        }
        a != b
    }
}

/// The side of `x` in the parity colouring from the root `r`: true when `x` is reached from
/// `r` by an odd walk that stays inside `s`.
define odd_walk_from[V](g: SimpleGraph[V], s: FiniteSet[V], r: V, x: V) -> Bool {
    exists(steps: List[V]) {
        simple_graph_walk(g, r, x, steps) and walk_in_set(s, steps) and nat_odd(steps.length)
    }
}

/// Every vertex of `s` is reachable from `r` by a walk that stays inside `s`.
define walk_reachable_from[V](g: SimpleGraph[V], s: FiniteSet[V], r: V) -> Bool {
    forall(x: V) {
        s.contains(x) implies exists(steps: List[V]) {
            simple_graph_walk(g, r, x, steps) and walk_in_set(s, steps)
        }
    }
}

/// Two walks from `r` to `x` of different parities that stay inside `s` join into an odd
/// closed walk.
theorem odd_walk_unique[V](
    g: SimpleGraph[V], s: FiniteSet[V], r: V, x: V, even_steps: List[V], odd_steps: List[V]
) {
    s.contains(r) and
    simple_graph_walk(g, r, x, even_steps) and walk_in_set(s, even_steps) and
        not nat_odd(even_steps.length) and
    simple_graph_walk(g, r, x, odd_steps) and walk_in_set(s, odd_steps) and
        nat_odd(odd_steps.length)
        implies has_odd_closed_walk(g, s)
} by {
    if s.contains(r) and
        simple_graph_walk(g, r, x, even_steps) and walk_in_set(s, even_steps) and
        not nat_odd(even_steps.length) and
        simple_graph_walk(g, r, x, odd_steps) and walk_in_set(s, odd_steps) and
        nat_odd(odd_steps.length) {
        simple_graph_walk_reverse(g, r, x, even_steps)
        simple_graph_walk(g, x, r, rev_walk_targets(even_steps, r))
        rev_walk_targets_in_set(s, even_steps, r)
        walk_in_set(s, rev_walk_targets(even_steps, r))
        simple_graph_walk_concat(g, x, r, x, rev_walk_targets(even_steps, r), odd_steps)
        simple_graph_walk(g, x, x, rev_walk_targets(even_steps, r) + odd_steps)
        walk_in_set_concat(s, rev_walk_targets(even_steps, r), odd_steps)
        walk_in_set(s, rev_walk_targets(even_steps, r) + odd_steps)
        rev_walk_targets_length(even_steps, r)
        rev_walk_targets(even_steps, r).length = even_steps.length
        add_length(rev_walk_targets(even_steps, r), odd_steps)
        rev_walk_targets(even_steps, r).length + odd_steps.length = (rev_walk_targets(even_steps, r) + odd_steps).length
        even_steps.length + odd_steps.length = (rev_walk_targets(even_steps, r) + odd_steps).length
        nat_odd_add(even_steps.length, odd_steps.length)
        nat_odd(even_steps.length + odd_steps.length) = (nat_odd(even_steps.length) != nat_odd(odd_steps.length))
        bool_ne_of_not_and(nat_odd(even_steps.length), nat_odd(odd_steps.length))
        not nat_odd(even_steps.length) and nat_odd(odd_steps.length) implies nat_odd(even_steps.length) != nat_odd(odd_steps.length)
        nat_odd(even_steps.length) != nat_odd(odd_steps.length)
        nat_odd(even_steps.length + odd_steps.length)
        nat_odd((rev_walk_targets(even_steps, r) + odd_steps).length)
        nat_odd_ne_zero((rev_walk_targets(even_steps, r) + odd_steps).length)
        (rev_walk_targets(even_steps, r) + odd_steps).length != Nat.0
        pos_of_ne_zero((rev_walk_targets(even_steps, r) + odd_steps).length)
        Nat.0 < (rev_walk_targets(even_steps, r) + odd_steps).length
        simple_graph_closed_walk_intro(g, x, rev_walk_targets(even_steps, r) + odd_steps)
        simple_graph_closed_walk(g, x, rev_walk_targets(even_steps, r) + odd_steps)
        has_odd_closed_walk(g, s) = exists(u: V, ws: List[V]) {
            simple_graph_closed_walk(g, u, ws) and nat_odd(ws.length) and walk_in_set(s, ws)
        }
        exists(u: V, ws: List[V]) {
            simple_graph_closed_walk(g, u, ws) and nat_odd(ws.length) and walk_in_set(s, ws)
        }
        has_odd_closed_walk(g, s)
    }
}

/// The parity colouring from a root is a bipartition of every edge of `s`.
theorem is_bipartition_of_parity_coloring[V](g: SimpleGraph[V], s: FiniteSet[V], r: V) {
    s.contains(r) and walk_reachable_from(g, s, r) and not has_odd_closed_walk(g, s) implies
        is_bipartition(g, s, odd_walk_from(g, s, r))
} by {
    if s.contains(r) and walk_reachable_from(g, s, r) and not has_odd_closed_walk(g, s) {
        is_bipartition_intro(g, s, odd_walk_from(g, s, r))
        forall(x: V, y: V) {
            if s.contains(x) and s.contains(y) and g.adj(x, y) {
                if odd_walk_from(g, s, r, x) {
                    odd_walk_from(g, s, r, x) = exists(steps: List[V]) {
                        simple_graph_walk(g, r, x, steps) and walk_in_set(s, steps) and nat_odd(steps.length)
                    }
                    exists(steps: List[V]) {
                        simple_graph_walk(g, r, x, steps) and walk_in_set(s, steps) and nat_odd(steps.length)
                    }
                    let (wx: List[V]) satisfy {
                        simple_graph_walk(g, r, x, wx) and walk_in_set(s, wx) and nat_odd(wx.length)
                    }
                    simple_graph_walk_append_adj(g, r, x, y, wx)
                    simple_graph_walk(g, r, y, wx.append(y))
                    walk_in_set_singleton(s, y)
                    walk_in_set(s, List.singleton(y))
                    walk_in_set_concat(s, wx, List.singleton(y))
                    walk_in_set(s, wx + List.singleton(y))
                    wx.append(y) = wx + List.singleton(y)
                    walk_in_set(s, wx.append(y))
                    add_length(wx, List.singleton(y))
                    wx.length + List.singleton(y).length = (wx + List.singleton(y)).length
                    List.singleton(y) = List.cons(y, List.nil[V])
                    List.cons(y, List.nil[V]).length = List.nil[V].length.suc
                    List.nil[V].length = Nat.0
                    List.singleton(y).length = Nat.1
                    wx.length + Nat.1 = (wx + List.singleton(y)).length
                    wx.append(y) = wx + List.singleton(y)
                    (wx.append(y)).length = wx.length + Nat.1
                    nat_odd_suc(wx.length)
                    nat_odd(wx.length.suc) = not nat_odd(wx.length)
                    nat_odd((wx.append(y)).length) = not nat_odd(wx.length)
                    not nat_odd((wx.append(y)).length)
                    if odd_walk_from(g, s, r, y) {
                        odd_walk_from(g, s, r, y) = exists(steps: List[V]) {
                            simple_graph_walk(g, r, y, steps) and walk_in_set(s, steps) and nat_odd(steps.length)
                        }
                        exists(steps: List[V]) {
                            simple_graph_walk(g, r, y, steps) and walk_in_set(s, steps) and nat_odd(steps.length)
                        }
                        let (wy: List[V]) satisfy {
                            simple_graph_walk(g, r, y, wy) and walk_in_set(s, wy) and nat_odd(wy.length)
                        }
                        odd_walk_unique(g, s, r, y, wx.append(y), wy)
                        has_odd_closed_walk(g, s)
                        false
                    }
                    not odd_walk_from(g, s, r, y)
                    bool_ne_of_and_not(odd_walk_from(g, s, r, x), odd_walk_from(g, s, r, y))
                    odd_walk_from(g, s, r, x) and not odd_walk_from(g, s, r, y) implies odd_walk_from(g, s, r, x) != odd_walk_from(g, s, r, y)
                    odd_walk_from(g, s, r, x) != odd_walk_from(g, s, r, y)
                }
                if not odd_walk_from(g, s, r, x) {
                    walk_reachable_from(g, s, r) = forall(z: V) {
                        s.contains(z) implies exists(steps: List[V]) {
                            simple_graph_walk(g, r, z, steps) and walk_in_set(s, steps)
                        }
                    }
                    forall(z: V) {
                        s.contains(z) implies exists(steps: List[V]) {
                            simple_graph_walk(g, r, z, steps) and walk_in_set(s, steps)
                        }
                    }
                    s.contains(x) implies exists(steps: List[V]) {
                        simple_graph_walk(g, r, x, steps) and walk_in_set(s, steps)
                    }
                    exists(steps: List[V]) {
                        simple_graph_walk(g, r, x, steps) and walk_in_set(s, steps)
                    }
                    let (wx: List[V]) satisfy {
                        simple_graph_walk(g, r, x, wx) and walk_in_set(s, wx)
                    }
                    if nat_odd(wx.length) {
                        odd_walk_from(g, s, r, x) = exists(steps: List[V]) {
                            simple_graph_walk(g, r, x, steps) and walk_in_set(s, steps) and nat_odd(steps.length)
                        }
                        exists(steps: List[V]) {
                            simple_graph_walk(g, r, x, steps) and walk_in_set(s, steps) and nat_odd(steps.length)
                        }
                        odd_walk_from(g, s, r, x)
                        false
                    }
                    not nat_odd(wx.length)
                    simple_graph_walk_append_adj(g, r, x, y, wx)
                    simple_graph_walk(g, r, y, wx.append(y))
                    walk_in_set_singleton(s, y)
                    walk_in_set(s, List.singleton(y))
                    walk_in_set_concat(s, wx, List.singleton(y))
                    walk_in_set(s, wx + List.singleton(y))
                    wx.append(y) = wx + List.singleton(y)
                    walk_in_set(s, wx.append(y))
                    add_length(wx, List.singleton(y))
                    wx.length + List.singleton(y).length = (wx + List.singleton(y)).length
                    List.singleton(y) = List.cons(y, List.nil[V])
                    List.cons(y, List.nil[V]).length = List.nil[V].length.suc
                    List.nil[V].length = Nat.0
                    List.singleton(y).length = Nat.1
                    wx.length + Nat.1 = (wx + List.singleton(y)).length
                    wx.append(y) = wx + List.singleton(y)
                    (wx.append(y)).length = wx.length + Nat.1
                    nat_odd_suc(wx.length)
                    nat_odd(wx.length.suc) = not nat_odd(wx.length)
                    nat_odd((wx.append(y)).length) = not nat_odd(wx.length)
                    nat_odd((wx.append(y)).length)
                    odd_walk_from(g, s, r, y) = exists(steps: List[V]) {
                        simple_graph_walk(g, r, y, steps) and walk_in_set(s, steps) and nat_odd(steps.length)
                    }
                    exists(steps: List[V]) {
                        simple_graph_walk(g, r, y, steps) and walk_in_set(s, steps) and nat_odd(steps.length)
                    }
                    odd_walk_from(g, s, r, y)
                    bool_ne_of_not_and(odd_walk_from(g, s, r, x), odd_walk_from(g, s, r, y))
                    not odd_walk_from(g, s, r, x) and odd_walk_from(g, s, r, y) implies odd_walk_from(g, s, r, x) != odd_walk_from(g, s, r, y)
                    odd_walk_from(g, s, r, x) != odd_walk_from(g, s, r, y)
                }
                odd_walk_from(g, s, r, x) != odd_walk_from(g, s, r, y)
            }
            s.contains(x) and s.contains(y) and g.adj(x, y) implies odd_walk_from(g, s, r, x) != odd_walk_from(g, s, r, y)
        }
        is_bipartition(g, s, odd_walk_from(g, s, r))
    }
}

/// A graph with no odd closed walk is bipartite: colour every vertex by the parity of the
/// length of a walk from a root, which is well defined because two walks of different parity
/// would join into an odd closed walk.
theorem is_bipartite_of_no_odd_closed_walk[V](g: SimpleGraph[V], s: FiniteSet[V], r: V) {
    s.contains(r) and walk_reachable_from(g, s, r) and not has_odd_closed_walk(g, s)
        implies is_bipartite(g, s)
} by {
    if s.contains(r) and walk_reachable_from(g, s, r) and not has_odd_closed_walk(g, s) {
        is_bipartition_of_parity_coloring(g, s, r)
        is_bipartition(g, s, odd_walk_from(g, s, r))
        is_bipartite_intro(g, s, odd_walk_from(g, s, r))
        is_bipartite(g, s)
    }
}

// The converse of `is_bipartite_no_odd_cycle` — that a graph with no odd cycle is bipartite —
// needs one more lemma: an odd closed walk contains an odd cycle (cut the walk at a repeated
// vertex and keep the shorter odd piece). The list-splitting machinery for that extraction is
// not yet in this file; with it, the theorem below follows from `is_bipartite_of_no_odd_closed_walk`.
//
// theorem is_bipartite_of_no_odd_cycle[V](g: SimpleGraph[V], s: FiniteSet[V], r: V) {
//     s.contains(r) and walk_reachable_from(g, s, r) and not has_odd_cycle(g, s)
//         implies is_bipartite(g, s)
// } by {
//     if s.contains(r) and walk_reachable_from(g, s, r) and not has_odd_cycle(g, s) {
//         // an odd closed walk would contain an odd cycle, so no odd cycle means no odd closed walk
//         is_bipartite_of_no_odd_closed_walk(g, s, r)
//         is_bipartite(g, s)
//     }
// }
