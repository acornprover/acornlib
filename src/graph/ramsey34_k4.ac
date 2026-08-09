from nat import Nat
from finite_set import FiniteSet
from data.basic.logic import exists_intro

/// True of the four distinct vertices of a red K4 in `s`.
define has_red_k4[V](color: (V, V) -> Bool, s: FiniteSet[V]) -> Bool {
    exists(w: V, x: V, y: V, z: V) {
        s.contains(w) and s.contains(x) and s.contains(y) and s.contains(z)
        and w != x and w != y and w != z and x != y and x != z and y != z
        and color(w, x) and color(w, y) and color(w, z)
        and color(x, y) and color(x, z) and color(y, z)
    }
}

/// Four vertices of `s` pairwise joined by red edges form a red K4.
theorem has_red_k4_intro[V](color: (V, V) -> Bool, s: FiniteSet[V], w: V, x: V, y: V, z: V) {
    s.contains(w) and s.contains(x) and s.contains(y) and s.contains(z)
    and w != x and w != y and w != z and x != y and x != z and y != z
    and color(w, x) and color(w, y) and color(w, z)
    and color(x, y) and color(x, z) and color(y, z)
    implies has_red_k4(color, s)
} by {
    if s.contains(w) and s.contains(x) and s.contains(y) and s.contains(z)
        and w != x and w != y and w != z and x != y and x != z and y != z
        and color(w, x) and color(w, y) and color(w, z)
        and color(x, y) and color(x, z) and color(y, z) {
        has_red_k4(color, s) = exists(a: V, b: V, c: V, d: V) {
            s.contains(a) and s.contains(b) and s.contains(c) and s.contains(d)
            and a != b and a != c and a != d and b != c and b != d and c != d
            and color(a, b) and color(a, c) and color(a, d)
            and color(b, c) and color(b, d) and color(c, d)
        }
        exists_intro(function(a: V) {
            exists(b: V, c: V, d: V) {
                s.contains(a) and s.contains(b) and s.contains(c) and s.contains(d)
                and a != b and a != c and a != d and b != c and b != d and c != d
                and color(a, b) and color(a, c) and color(a, d)
                and color(b, c) and color(b, d) and color(c, d)
            }
        }, w)
        exists_intro(function(b: V) {
            exists(c: V, d: V) {
                s.contains(w) and s.contains(b) and s.contains(c) and s.contains(d)
                and w != b and w != c and w != d and b != c and b != d and c != d
                and color(w, b) and color(w, c) and color(w, d)
                and color(b, c) and color(b, d) and color(c, d)
            }
        }, x)
        exists_intro(function(c: V) {
            exists(d: V) {
                s.contains(w) and s.contains(x) and s.contains(c) and s.contains(d)
                and w != x and w != c and w != d and x != c and x != d and c != d
                and color(w, x) and color(w, c) and color(w, d)
                and color(x, c) and color(x, d) and color(c, d)
            }
        }, y)
        exists_intro(function(d: V) {
            s.contains(w) and s.contains(x) and s.contains(y) and s.contains(d)
            and w != x and w != y and w != d and x != y and x != d and y != d
            and color(w, x) and color(w, y) and color(w, d)
            and color(x, y) and color(x, d) and color(y, d)
        }, z)
        exists(a: V, b: V, c: V, d: V) {
            s.contains(a) and s.contains(b) and s.contains(c) and s.contains(d)
            and a != b and a != c and a != d and b != c and b != d and c != d
            and color(a, b) and color(a, c) and color(a, d)
            and color(b, c) and color(b, d) and color(c, d)
        }
        has_red_k4(color, s)
    }
}

/// True of the four distinct vertices of a blue K4 in `s`.
define has_blue_k4[V](color: (V, V) -> Bool, s: FiniteSet[V]) -> Bool {
    exists(w: V, x: V, y: V, z: V) {
        s.contains(w) and s.contains(x) and s.contains(y) and s.contains(z)
        and w != x and w != y and w != z and x != y and x != z and y != z
        and not color(w, x) and not color(w, y) and not color(w, z)
        and not color(x, y) and not color(x, z) and not color(y, z)
    }
}

/// Four vertices of `s` pairwise joined by blue edges form a blue K4.
theorem has_blue_k4_intro[V](color: (V, V) -> Bool, s: FiniteSet[V], w: V, x: V, y: V, z: V) {
    s.contains(w) and s.contains(x) and s.contains(y) and s.contains(z)
    and w != x and w != y and w != z and x != y and x != z and y != z
    and not color(w, x) and not color(w, y) and not color(w, z)
    and not color(x, y) and not color(x, z) and not color(y, z)
    implies has_blue_k4(color, s)
} by {
    if s.contains(w) and s.contains(x) and s.contains(y) and s.contains(z)
        and w != x and w != y and w != z and x != y and x != z and y != z
        and not color(w, x) and not color(w, y) and not color(w, z)
        and not color(x, y) and not color(x, z) and not color(y, z) {
        has_blue_k4(color, s) = exists(a: V, b: V, c: V, d: V) {
            s.contains(a) and s.contains(b) and s.contains(c) and s.contains(d)
            and a != b and a != c and a != d and b != c and b != d and c != d
            and not color(a, b) and not color(a, c) and not color(a, d)
            and not color(b, c) and not color(b, d) and not color(c, d)
        }
        exists_intro(function(a: V) {
            exists(b: V, c: V, d: V) {
                s.contains(a) and s.contains(b) and s.contains(c) and s.contains(d)
                and a != b and a != c and a != d and b != c and b != d and c != d
                and not color(a, b) and not color(a, c) and not color(a, d)
                and not color(b, c) and not color(b, d) and not color(c, d)
            }
        }, w)
        exists_intro(function(b: V) {
            exists(c: V, d: V) {
                s.contains(w) and s.contains(b) and s.contains(c) and s.contains(d)
                and w != b and w != c and w != d and b != c and b != d and c != d
                and not color(w, b) and not color(w, c) and not color(w, d)
                and not color(b, c) and not color(b, d) and not color(c, d)
            }
        }, x)
        exists_intro(function(c: V) {
            exists(d: V) {
                s.contains(w) and s.contains(x) and s.contains(c) and s.contains(d)
                and w != x and w != c and w != d and x != c and x != d and c != d
                and not color(w, x) and not color(w, c) and not color(w, d)
                and not color(x, c) and not color(x, d) and not color(c, d)
            }
        }, y)
        exists_intro(function(d: V) {
            s.contains(w) and s.contains(x) and s.contains(y) and s.contains(d)
            and w != x and w != y and w != d and x != y and x != d and y != d
            and not color(w, x) and not color(w, y) and not color(w, d)
            and not color(x, y) and not color(x, d) and not color(y, d)
        }, z)
        exists(a: V, b: V, c: V, d: V) {
            s.contains(a) and s.contains(b) and s.contains(c) and s.contains(d)
            and a != b and a != c and a != d and b != c and b != d and c != d
            and not color(a, b) and not color(a, c) and not color(a, d)
            and not color(b, c) and not color(b, d) and not color(c, d)
        }
        has_blue_k4(color, s)
    }
}

/// True if some K4 of `s` is monochromatic: all red or all blue.
define has_mono_k4[V](color: (V, V) -> Bool, s: FiniteSet[V]) -> Bool {
    has_red_k4(color, s) or has_blue_k4(color, s)
}
