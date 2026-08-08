from nat import Nat, from_nat, pow_zero, pow_one
from list import List, sum, map, is_permutation, permutation_preserves_mapped_sum,
    map_map, map_sum_add, sum_map_of_pointwise
from algebra.add_comm_monoid import AddCommMonoid
from semiring import Semiring
from data.basic.functions import compose
from algebra.add_semigroup import add_fn
from algebra.semigroup import mul_fn
from number_theory import coprime_residues, coprime_residues_below, coprime_residues_length,
    coprime_residues_contains_imp
from reduced_residue_reflection import reflect_residue, reflect_permutes_coprime_residues,
    reflect_residue_adds_to_modulus

numerals Nat

/// The sum of a function over the naturals below `k` that are coprime to `n`.
///
/// The truncated form, needed because the reduced residues are built up one candidate at a
/// time and every proof about the sum runs along that construction.
define reduced_residue_sum_below[A: AddCommMonoid](
    n: Nat, k: Nat, f: Nat -> A
) -> A {
    sum(map(coprime_residues_below(n, k), f))
}

/// The sum of a function over the reduced residues modulo `n`.
///
/// The general sum the rest of this file specialises: the power sums, the totient itself, and
/// character sums are all this with a particular summand.
define reduced_residue_sum[A: AddCommMonoid](n: Nat, f: Nat -> A) -> A {
    reduced_residue_sum_below(n, n, f)
}

/// The empty range contributes nothing.
theorem reduced_residue_sum_below_zero[A: AddCommMonoid](n: Nat, f: Nat -> A) {
    reduced_residue_sum_below(n, Nat.0, f) = A.0
} by {
    coprime_residues_below(n, Nat.0) = List.nil[Nat]
    map(List.nil[Nat], f) = List.nil[A]
    sum(List.nil[A]) = A.0
}

/// A coprime candidate adds its value.
theorem reduced_residue_sum_below_suc_yes[A: AddCommMonoid](
    n: Nat, j: Nat, f: Nat -> A
) {
    j.coprime(n)
        implies reduced_residue_sum_below(n, j.suc, f) = f(j) + reduced_residue_sum_below(n, j, f)
} by {
    if j.coprime(n) {
        coprime_residues_below(n, j.suc) = List.cons(j, coprime_residues_below(n, j))
        (map(List.cons(j, coprime_residues_below(n, j)), f)
            = List.cons(f(j), map(coprime_residues_below(n, j), f)))
        (sum(List.cons(f(j), map(coprime_residues_below(n, j), f)))
            = f(j) + sum(map(coprime_residues_below(n, j), f)))
        reduced_residue_sum_below(n, j.suc, f) = f(j) + reduced_residue_sum_below(n, j, f)
    }
}

/// A candidate that is not coprime contributes nothing.
theorem reduced_residue_sum_below_suc_no[A: AddCommMonoid](
    n: Nat, j: Nat, f: Nat -> A
) {
    not j.coprime(n)
        implies reduced_residue_sum_below(n, j.suc, f) = reduced_residue_sum_below(n, j, f)
} by {
    if not j.coprime(n) {
        coprime_residues_below(n, j.suc) = coprime_residues_below(n, j)
        reduced_residue_sum_below(n, j.suc, f) = reduced_residue_sum_below(n, j, f)
    }
}

/// Summands agreeing on the reduced residues give equal sums.
///
/// The sum sees the summand only at coprime arguments, which is what lets one be replaced by
/// any convenient description of it there.
theorem reduced_residue_sum_below_congr[A: AddCommMonoid](
    n: Nat, k: Nat, f: Nat -> A, g: Nat -> A
) {
    (forall(a: Nat) { a.coprime(n) implies f(a) = g(a) })
        implies reduced_residue_sum_below(n, k, f) = reduced_residue_sum_below(n, k, g)
} by {
    if forall(a: Nat) { a.coprime(n) implies f(a) = g(a) } {
        define p(x: Nat) -> Bool {
            reduced_residue_sum_below(n, x, f) = reduced_residue_sum_below(n, x, g)
        }
        reduced_residue_sum_below_zero(n, f)
        reduced_residue_sum_below_zero(n, g)
        reduced_residue_sum_below(n, Nat.0, f) = reduced_residue_sum_below(n, Nat.0, g)
        p(Nat.0)
        forall(j: Nat) {
            if p(j) {
                reduced_residue_sum_below(n, j, f) = reduced_residue_sum_below(n, j, g)
                if j.coprime(n) {
                    f(j) = g(j)
                    reduced_residue_sum_below_suc_yes(n, j, f)
                    reduced_residue_sum_below_suc_yes(n, j, g)
                    (reduced_residue_sum_below(n, j.suc, f)
                        = reduced_residue_sum_below(n, j.suc, g))
                }
                if not j.coprime(n) {
                    reduced_residue_sum_below_suc_no(n, j, f)
                    reduced_residue_sum_below_suc_no(n, j, g)
                    (reduced_residue_sum_below(n, j.suc, f)
                        = reduced_residue_sum_below(n, j.suc, g))
                }
                (reduced_residue_sum_below(n, j.suc, f)
                    = reduced_residue_sum_below(n, j.suc, g))
                p(j.suc)
            }
            (p(j) implies p(j.suc))
        }
        p(Nat.0) and forall(j: Nat) {
            p(j) implies p(j.suc)
        }
        Nat.induction(p)
        p(k)
        reduced_residue_sum_below(n, k, f) = reduced_residue_sum_below(n, k, g)
    }
}

/// Summands agreeing on the reduced residues give equal reduced residue sums.
theorem reduced_residue_sum_congr[A: AddCommMonoid](
    n: Nat, f: Nat -> A, g: Nat -> A
) {
    (forall(a: Nat) { a.coprime(n) implies f(a) = g(a) })
        implies reduced_residue_sum(n, f) = reduced_residue_sum(n, g)
} by {
    if forall(a: Nat) { a.coprime(n) implies f(a) = g(a) } {
        reduced_residue_sum_below_congr(n, n, f, g)
        reduced_residue_sum_below(n, n, f) = reduced_residue_sum_below(n, n, g)
        reduced_residue_sum(n, f) = reduced_residue_sum(n, g)
    }
}

/// The summand that is one at every argument.
define one_summand(a: Nat) -> Nat {
    Nat.1
}

/// Summing one over the truncated reduced residues counts them.
theorem reduced_residue_sum_below_one(n: Nat, k: Nat) {
    reduced_residue_sum_below(n, k, one_summand) = coprime_residues_below(n, k).length
} by {
    define p(x: Nat) -> Bool {
        reduced_residue_sum_below(n, x, one_summand) = coprime_residues_below(n, x).length
    }
    reduced_residue_sum_below_zero(n, one_summand)
    coprime_residues_below(n, Nat.0) = List.nil[Nat]
    List.nil[Nat].length = Nat.0
    p(Nat.0)
    forall(j: Nat) {
        if p(j) {
            (reduced_residue_sum_below(n, j, one_summand)
                = coprime_residues_below(n, j).length)
            if j.coprime(n) {
                reduced_residue_sum_below_suc_yes(n, j, one_summand)
                one_summand(j) = Nat.1
                (reduced_residue_sum_below(n, j.suc, one_summand)
                    = Nat.1 + coprime_residues_below(n, j).length)
                coprime_residues_below(n, j.suc) = List.cons(j, coprime_residues_below(n, j))
                (List.cons(j, coprime_residues_below(n, j)).length
                    = coprime_residues_below(n, j).length + Nat.1)
                (Nat.1 + coprime_residues_below(n, j).length
                    = coprime_residues_below(n, j).length + Nat.1)
                (reduced_residue_sum_below(n, j.suc, one_summand)
                    = coprime_residues_below(n, j.suc).length)
            }
            if not j.coprime(n) {
                reduced_residue_sum_below_suc_no(n, j, one_summand)
                coprime_residues_below(n, j.suc) = coprime_residues_below(n, j)
                (reduced_residue_sum_below(n, j.suc, one_summand)
                    = coprime_residues_below(n, j.suc).length)
            }
            (reduced_residue_sum_below(n, j.suc, one_summand)
                = coprime_residues_below(n, j.suc).length)
            p(j.suc)
        }
        (p(j) implies p(j.suc))
    }
    p(Nat.0) and forall(j: Nat) {
        p(j) implies p(j.suc)
    }
    Nat.induction(p)
    p(k)
    reduced_residue_sum_below(n, k, one_summand) = coprime_residues_below(n, k).length
}

/// Summing one over the reduced residues gives the totient.
///
/// The first check that the general sum is the right notion: at the constant summand it
/// recovers the count the reduced residues were introduced to express.
theorem reduced_residue_sum_one(n: Nat) {
    reduced_residue_sum(n, one_summand) = n.totient
} by {
    reduced_residue_sum_below_one(n, n)
    reduced_residue_sum_below(n, n, one_summand) = coprime_residues_below(n, n).length
    coprime_residues(n) = coprime_residues_below(n, n)
    coprime_residues_length(n)
    coprime_residues(n).length = n.totient
    reduced_residue_sum(n, one_summand) = n.totient
}

/// The summand raising its argument to a fixed power.
define power_summand[S: Semiring](k: Nat, a: Nat) -> S {
    from_nat[S](a).pow(k)
}

/// The power sum of the reduced residues modulo `n`.
///
/// Written `phi_k(n)` in the literature. The zeroth is the totient and the first has the
/// closed form `n * totient(n) / 2` for `n > 1`, by pairing each residue with its reflection.
define reduced_residue_power_sum[S: Semiring](k: Nat, n: Nat) -> S {
    reduced_residue_sum(n, power_summand[S](k))
}

/// The zeroth power summand is one.
theorem power_summand_zero[S: Semiring](a: Nat) {
    power_summand[S](Nat.0, a) = S.1
} by {
    pow_zero(from_nat[S](a))
    from_nat[S](a).pow(Nat.0) = S.1
}

/// The first power summand is the argument itself.
theorem power_summand_one[S: Semiring](a: Nat) {
    power_summand[S](Nat.1, a) = from_nat[S](a)
} by {
    pow_one(from_nat[S](a))
    from_nat[S](a).pow(Nat.1) = from_nat[S](a)
}

/// The summand that is one at every argument, in the semiring.
define one_summand_in[S: Semiring](a: Nat) -> S {
    S.1
}

/// The zeroth power sum counts the reduced residues.
///
/// Every summand is one, so the sum is the sum of the constant one, which is the totient.
theorem reduced_residue_power_sum_zero[S: Semiring](n: Nat) {
    reduced_residue_power_sum[S](Nat.0, n) = reduced_residue_sum(n, one_summand_in[S])
} by {
    forall(a: Nat) {
        power_summand_zero[S](a)
        power_summand[S](Nat.0, a) = S.1
        one_summand_in[S](a) = S.1
        power_summand[S](Nat.0, a) = one_summand_in[S](a)
        (a.coprime(n) implies power_summand[S](Nat.0, a) = one_summand_in[S](a))
    }
    reduced_residue_sum_congr(n, power_summand[S](Nat.0), one_summand_in[S])
    (reduced_residue_sum(n, power_summand[S](Nat.0))
        = reduced_residue_sum(n, one_summand_in[S]))
    reduced_residue_power_sum[S](Nat.0, n) = reduced_residue_sum(n, one_summand_in[S])
}

/// A reduced residue sum is invariant under any reordering of the residue list.
///
/// The permutation is a hypothesis rather than something derived here: the facts that would
/// discharge it for a concrete reordering, such as `coprime_residues_all_coprime` and
/// `coprime_residues_unique`, exist in `src/number_theory/totient.ac` but are not part of the
/// exported `number_theory` interface. Stated this way the result is usable now and becomes
/// automatic once they are exported.
theorem reduced_residue_sum_permuted[A: AddCommMonoid](
    n: Nat, f: Nat -> A, reordered: List[Nat]
) {
    is_permutation(coprime_residues(n), reordered)
        implies reduced_residue_sum(n, f) = sum(map(reordered, f))
} by {
    permutation_preserves_mapped_sum(coprime_residues(n), reordered, f)
    sum(map(coprime_residues(n), f)) = sum(map(reordered, f))
    coprime_residues(n) = coprime_residues_below(n, n)
    reduced_residue_sum(n, f) = sum(map(coprime_residues_below(n, n), f))
    reduced_residue_sum(n, f) = sum(map(coprime_residues(n), f))
    (is_permutation(coprime_residues(n), reordered)
        implies reduced_residue_sum(n, f) = sum(map(reordered, f)))
}

/// Reordering the residue list by a map leaves the sum unchanged.
///
/// The form the reflection argument needs: if applying `g` to every reduced residue permutes
/// the list, then summing `f` after `g` over the residues gives the same total as summing `f`.
theorem reduced_residue_sum_reindex[A: AddCommMonoid](
    n: Nat, f: Nat -> A, g: Nat -> Nat
) {
    is_permutation(coprime_residues(n), map(coprime_residues(n), g))
        implies reduced_residue_sum(n, f) = reduced_residue_sum(n, compose(f, g))
} by {
    if is_permutation(coprime_residues(n), map(coprime_residues(n), g)) {
        reduced_residue_sum_permuted(n, f, map(coprime_residues(n), g))
        reduced_residue_sum(n, f) = sum(map(map(coprime_residues(n), g), f))
        map(map(coprime_residues(n), g), f) = map(coprime_residues(n), compose(f, g))
        reduced_residue_sum(n, f) = sum(map(coprime_residues(n), compose(f, g)))
        coprime_residues(n) = coprime_residues_below(n, n)
        (reduced_residue_sum(n, compose(f, g))
            = sum(map(coprime_residues_below(n, n), compose(f, g))))
        reduced_residue_sum(n, f) = reduced_residue_sum(n, compose(f, g))
    }
}

/// The summand that returns its argument.
define id_summand(a: Nat) -> Nat {
    a
}

/// The summand that is the modulus at every argument.
define modulus_summand(n: Nat, a: Nat) -> Nat {
    n
}

/// Summing the modulus over the reduced residues gives the modulus times their count.
theorem reduced_residue_sum_below_modulus(n: Nat, k: Nat) {
    reduced_residue_sum_below(n, k, modulus_summand(n))
        = n * coprime_residues_below(n, k).length
} by {
    define p(x: Nat) -> Bool {
        reduced_residue_sum_below(n, x, modulus_summand(n))
            = n * coprime_residues_below(n, x).length
    }
    reduced_residue_sum_below_zero(n, modulus_summand(n))
    coprime_residues_below(n, Nat.0) = List.nil[Nat]
    List.nil[Nat].length = Nat.0
    n * Nat.0 = Nat.0
    p(Nat.0)
    forall(j: Nat) {
        if p(j) {
            (reduced_residue_sum_below(n, j, modulus_summand(n))
                = n * coprime_residues_below(n, j).length)
            if j.coprime(n) {
                reduced_residue_sum_below_suc_yes(n, j, modulus_summand(n))
                modulus_summand(n, j) = n
                (reduced_residue_sum_below(n, j.suc, modulus_summand(n))
                    = n + n * coprime_residues_below(n, j).length)
                coprime_residues_below(n, j.suc) = List.cons(j, coprime_residues_below(n, j))
                (coprime_residues_below(n, j.suc).length
                    = coprime_residues_below(n, j).length + Nat.1)
                (n * (coprime_residues_below(n, j).length + Nat.1)
                    = n * coprime_residues_below(n, j).length + n * Nat.1)
                n * Nat.1 = n
                (n * coprime_residues_below(n, j).length + n
                    = n + n * coprime_residues_below(n, j).length)
                (reduced_residue_sum_below(n, j.suc, modulus_summand(n))
                    = n * coprime_residues_below(n, j.suc).length)
            }
            if not j.coprime(n) {
                reduced_residue_sum_below_suc_no(n, j, modulus_summand(n))
                coprime_residues_below(n, j.suc) = coprime_residues_below(n, j)
                (reduced_residue_sum_below(n, j.suc, modulus_summand(n))
                    = n * coprime_residues_below(n, j.suc).length)
            }
            (reduced_residue_sum_below(n, j.suc, modulus_summand(n))
                = n * coprime_residues_below(n, j.suc).length)
            p(j.suc)
        }
        (p(j) implies p(j.suc))
    }
    p(Nat.0) and forall(j: Nat) {
        p(j) implies p(j.suc)
    }
    Nat.induction(p)
    p(k)
    (reduced_residue_sum_below(n, k, modulus_summand(n))
        = n * coprime_residues_below(n, k).length)
}

/// Summing the modulus over the reduced residues gives the modulus times their count.
theorem reduced_residue_sum_modulus(n: Nat) {
    reduced_residue_sum(n, modulus_summand(n)) = n * n.totient
} by {
    reduced_residue_sum_below_modulus(n, n)
    (reduced_residue_sum_below(n, n, modulus_summand(n))
        = n * coprime_residues_below(n, n).length)
    coprime_residues(n) = coprime_residues_below(n, n)
    coprime_residues_length(n)
    coprime_residues(n).length = n.totient
    coprime_residues_below(n, n).length = n.totient
    reduced_residue_sum(n, modulus_summand(n)) = n * n.totient
}

/// The reduced residues pair with their reflections to give twice the first power sum.
///
/// Each residue `a` pairs with `n - a`, which is again a reduced residue, and the pair sums to
/// the modulus. Over all `totient(n)` residues that is `n * totient(n)`, and it counts the
/// first power sum twice.
///
/// Stated in the doubled form rather than as `phi_1(n) = n * totient(n) / 2`, because dividing
/// by two is not available over the naturals without a separate argument that the total is
/// even. The doubled identity is the content; the halved form follows in any ring where two is
/// invertible.
theorem reduced_residue_first_power_sum_doubled(n: Nat) {
    Nat.1 < n implies
        reduced_residue_sum(n, id_summand) + reduced_residue_sum(n, id_summand) = n * n.totient
} by {
    if Nat.1 < n {
        reflect_permutes_coprime_residues(n)
        is_permutation(coprime_residues(n), map(coprime_residues(n), reflect_residue(n)))
        reduced_residue_sum_reindex(n, id_summand, reflect_residue(n))
        (reduced_residue_sum(n, id_summand)
            = reduced_residue_sum(n, compose(id_summand, reflect_residue(n))))
        coprime_residues(n) = coprime_residues_below(n, n)
        (reduced_residue_sum(n, id_summand) = sum(map(coprime_residues(n), id_summand)))
        (reduced_residue_sum(n, compose(id_summand, reflect_residue(n)))
            = sum(map(coprime_residues(n), compose(id_summand, reflect_residue(n)))))
        map_sum_add(coprime_residues(n), id_summand, compose(id_summand, reflect_residue(n)))
        (sum(map(coprime_residues(n), id_summand))
            + sum(map(coprime_residues(n), compose(id_summand, reflect_residue(n))))
            = sum(map(coprime_residues(n),
                add_fn(id_summand, compose(id_summand, reflect_residue(n))))))
        forall(a: Nat) {
            if coprime_residues(n).contains(a) {
                coprime_residues_contains_imp(n, a)
                a < n
                a <= n
                reflect_residue_adds_to_modulus(n, a)
                a + reflect_residue(n, a) = n
                id_summand(a) = a
                compose(id_summand, reflect_residue(n))(a) = id_summand(reflect_residue(n, a))
                id_summand(reflect_residue(n, a)) = reflect_residue(n, a)
                (add_fn(id_summand, compose(id_summand, reflect_residue(n)), a)
                    = a + reflect_residue(n, a))
                (add_fn(id_summand, compose(id_summand, reflect_residue(n)), a)
                    = modulus_summand(n, a))
            }
            (coprime_residues(n).contains(a) implies
                add_fn(id_summand, compose(id_summand, reflect_residue(n)), a)
                    = modulus_summand(n, a))
        }
        sum_map_of_pointwise(coprime_residues(n),
            add_fn(id_summand, compose(id_summand, reflect_residue(n))), modulus_summand(n))
        (sum(map(coprime_residues(n),
            add_fn(id_summand, compose(id_summand, reflect_residue(n)))))
            = sum(map(coprime_residues(n), modulus_summand(n))))
        reduced_residue_sum_modulus(n)
        (reduced_residue_sum(n, modulus_summand(n))
            = sum(map(coprime_residues(n), modulus_summand(n))))
        sum(map(coprime_residues(n), modulus_summand(n))) = n * n.totient
        (reduced_residue_sum(n, id_summand) + reduced_residue_sum(n, id_summand)
            = n * n.totient)
    }
}
