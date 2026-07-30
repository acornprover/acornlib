from nat import Nat
from int import Int
from rat import Rat, from_nat_add, nat_lt_imp_rat_lt, mul_inv_cancels_right
from list import List
from pair import Pair
from residue_class_density import modulus_weight, system_density, system_density_nil,
    system_density_cons

numerals Nat

/// The system of the first `k` residue classes modulo `m`.
///
/// Built downward from `k`, so that `full_residue_system(m, m)` is the system of every residue
/// modulo `m` and the recursion peels off one class at a time.
define full_residue_system(m: Nat, k: Nat) -> List[Pair[Nat, Nat]] {
    match k {
        Nat.zero {
            List.nil[Pair[Nat, Nat]]
        }
        Nat.suc(pred) {
            List.cons(Pair.new(m, pred), full_residue_system(m, pred))
        }
    }
}

/// The empty case of the construction.
theorem full_residue_system_zero(m: Nat) {
    full_residue_system(m, Nat.0) = List.nil[Pair[Nat, Nat]]
}

/// One more class is added at the front.
theorem full_residue_system_suc(m: Nat, k: Nat) {
    full_residue_system(m, k.suc)
        = List.cons(Pair.new(m, k), full_residue_system(m, k))
}

/// Every class of the construction has the same modulus, so the density is a multiple of one
/// weight.
///
/// Induction along the construction: each class contributes the reciprocal of `m`, and there are
/// `k` of them.
theorem full_residue_system_density(m: Nat, k: Nat) {
    system_density(full_residue_system(m, k)) = Rat.from_nat(k) * modulus_weight(m)
} by {
    define p(x: Nat) -> Bool {
        system_density(full_residue_system(m, x)) = Rat.from_nat(x) * modulus_weight(m)
    }
    full_residue_system_zero(m)
    system_density_nil
    (system_density(List.nil[Pair[Nat, Nat]]) = Rat.0)
    (Rat.from_nat(Nat.0) = Rat.0)
    (Rat.0 * modulus_weight(m) = Rat.0)
    p(Nat.0)
    forall(x: Nat) {
        if p(x) {
            (system_density(full_residue_system(m, x))
                = Rat.from_nat(x) * modulus_weight(m))
            full_residue_system_suc(m, x)
            system_density_cons(Pair.new(m, x), full_residue_system(m, x))
            (system_density(List.cons(Pair.new(m, x), full_residue_system(m, x)))
                = modulus_weight(Pair.new(m, x).first)
                    + system_density(full_residue_system(m, x)))
            (Pair.new(m, x).first = m)
            (system_density(full_residue_system(m, x.suc))
                = modulus_weight(m) + Rat.from_nat(x) * modulus_weight(m))
            (Rat.1 * modulus_weight(m) = modulus_weight(m))
            (Rat.1 * modulus_weight(m) + Rat.from_nat(x) * modulus_weight(m)
                = (Rat.1 + Rat.from_nat(x)) * modulus_weight(m))
            (Rat.from_nat(Nat.1) = Rat.1)
            from_nat_add(Nat.1, x)
            (Rat.from_nat(Nat.1) + Rat.from_nat(x) = Rat.from_nat(Nat.1 + x))
            (Nat.1 + x = x.suc)
            (Rat.1 + Rat.from_nat(x) = Rat.from_nat(x.suc))
            (system_density(full_residue_system(m, x.suc))
                = Rat.from_nat(x.suc) * modulus_weight(m))
            p(x.suc)
        }
        (p(x) implies p(x.suc))
    }
    p(Nat.0) and forall(x: Nat) {
        p(x) implies p(x.suc)
    }
    Nat.induction(p)
    p(k)
}

/// The system of every residue modulo a positive `m` has density exactly one.
///
/// The explicit construction attaining the density a covering and disjoint system must have. It
/// needs no covering or disjointness argument: the weights are all the same and there are exactly
/// as many classes as the modulus.
theorem full_residue_system_density_one(m: Nat) {
    Nat.0 < m implies system_density(full_residue_system(m, m)) = Rat.1
} by {
    if Nat.0 < m {
        full_residue_system_density(m, m)
        (system_density(full_residue_system(m, m)) = Rat.from_nat(m) * modulus_weight(m))
        nat_lt_imp_rat_lt(Nat.0, m)
        (Rat.from_nat(Nat.0) < Rat.from_nat(m))
        (Rat.from_nat(Nat.0) = Rat.0)
        Rat.from_nat(m) != Rat.0
        (Rat.from_nat(m) = Rat.from_int(Int.from_nat(m)))
        (modulus_weight(m) = Rat.from_int(Int.from_nat(m)).inverse)
        (modulus_weight(m) = Rat.from_nat(m).inverse)
        mul_inv_cancels_right(Rat.from_nat(m))
        (Rat.from_nat(m) * Rat.from_nat(m).inverse = Rat.1)
        system_density(full_residue_system(m, m)) = Rat.1
    }
}
