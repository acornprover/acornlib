from algebra.add_comm_monoid import AddCommMonoid
from algebra.add_semigroup import add_fn
from finite_set import FiniteSet, fs_from_list, fs_insert, fs_union,
    finite_set_has_unique_list, finite_set_sum, finite_set_sum_eq_list_sum,
    finite_set_sum_insert, finite_set_sum_add, finite_set_sum_disjoint_union,
    finite_set_sum_scalar_mul
from data.finite.finite_set_product import finite_set_product, finite_set_product_contains_eq,
    finite_set_product_contains_pair, finite_set_product_contains_left,
    finite_set_product_contains_right, finite_set_product_subset
from data.finite.finite_set_sum_product import finite_set_product_row_fn,
    finite_set_sum_product_fubini_rows
from data.finite.finite_set_sum_real import pointwise_le, pointwise_le_at, finite_set_sum_le,
    finite_set_sum_nonneg
from list import List, map, sum, sum_map_of_pointwise
from real import Real
from pair import Pair
from algebra.semigroup import mul_fn
from semiring import Semiring

/// Transport a finite-set sum across pointwise equality on the finite set.
theorem finite_set_sum_eq_of_pointwise_on_set[T, A: AddCommMonoid](
    s: FiniteSet[T], f: T -> A, g: T -> A
) {
    (forall(x: T) { s.contains(x) implies f(x) = g(x) }) implies
        finite_set_sum(s, f) = finite_set_sum(s, g)
} by {
    if forall(x: T) { s.contains(x) implies f(x) = g(x) } {
        finite_set_has_unique_list(s)
        let items: List[T] satisfy {
            fs_from_list(items) = s and items.is_unique
        }
        finite_set_sum_eq_list_sum[T, A](items, f)
        finite_set_sum_eq_list_sum[T, A](items, g)
        finite_set_sum(s, f) = sum[A](map(items, f))
        finite_set_sum(s, g) = sum[A](map(items, g))
        forall(x: T) {
            if items.contains(x) {
                s.contains(x)
                f(x) = g(x)
            }
        }
        sum_map_of_pointwise[T, A](items, f, g)
        sum[A](map(items, f)) = sum[A](map(items, g))
        finite_set_sum(s, f) = finite_set_sum(s, g)
    }
}

/// Finite-set sums over equal functions are equal.
theorem finite_set_sum_eq_of_function_eq_global[T, A: AddCommMonoid](
    s: FiniteSet[T], f: T -> A, g: T -> A
) {
    f = g implies finite_set_sum(s, f) = finite_set_sum(s, g)
} by {
    if f = g {
        forall(x: T) {
            if s.contains(x) {
                f(x) = g(x)
            }
        }
        finite_set_sum_eq_of_pointwise_on_set(s, f, g)
        finite_set_sum(s, f) = finite_set_sum(s, g)
    }
}

/// The finite-set sum of a pointwise sum agrees with the sum of the finite-set sums.
theorem finite_set_sum_pointwise_add[T, A: AddCommMonoid](s: FiniteSet[T], f: T -> A, g: T -> A) {
    finite_set_sum(s, add_fn(f, g)) = finite_set_sum(s, f) + finite_set_sum(s, g)
} by {
    finite_set_sum_add(s, f, g)
    finite_set_sum(s, f) + finite_set_sum(s, g) = finite_set_sum(s, add_fn(f, g))
}

/// Inserting a new element can also be read as adding its value on the right.
theorem finite_set_sum_insert_right[T, A: AddCommMonoid](s: FiniteSet[T], item: T, f: T -> A) {
    not s.contains(item) implies finite_set_sum(fs_insert(s, item), f) = finite_set_sum(s, f) + f(item)
} by {
    if not s.contains(item) {
        finite_set_sum_insert(s, item, f)
        finite_set_sum(fs_insert(s, item), f) = f(item) + finite_set_sum(s, f)
        f(item) + finite_set_sum(s, f) = finite_set_sum(s, f) + f(item)
        finite_set_sum(fs_insert(s, item), f) = finite_set_sum(s, f) + f(item)
    }
}

/// The disjoint-union finite-set sum can also be read with the summands swapped.
theorem finite_set_sum_disjoint_union_commuted[T, A: AddCommMonoid](
    a: FiniteSet[T], b: FiniteSet[T], f: T -> A
) {
    a.is_disjoint(b) implies
        finite_set_sum(fs_union(a, b), f) = finite_set_sum(b, f) + finite_set_sum(a, f)
} by {
    if a.is_disjoint(b) {
        finite_set_sum_disjoint_union(a, b, f)
        finite_set_sum(fs_union(a, b), f) = finite_set_sum(a, f) + finite_set_sum(b, f)
        finite_set_sum(a, f) + finite_set_sum(b, f) = finite_set_sum(b, f) + finite_set_sum(a, f)
        finite_set_sum(fs_union(a, b), f) = finite_set_sum(b, f) + finite_set_sum(a, f)
    }
}

/// Scalar multiplication of a finite-set sum, with the finite-set sum on the left.
theorem finite_set_sum_scalar_mul_right_form[T, S: Semiring](c: S, s: FiniteSet[T], f: T -> S) {
    finite_set_sum(s, mul_fn(c, f)) = c * finite_set_sum(s, f)
} by {
    finite_set_sum_scalar_mul(c, s, f)
    c * finite_set_sum(s, f) = finite_set_sum(s, mul_fn(c, f))
}

/// Finite-product membership is equivalent to component membership, as an implication pair.
theorem finite_set_product_contains_iff_components[T, U](
    a: FiniteSet[T], b: FiniteSet[U], p: Pair[T, U]
) {
    finite_set_product(a, b).contains(p) iff (a.contains(p.first) and b.contains(p.second))
} by {
    finite_set_product_contains_eq(a, b, p)
    finite_set_product(a, b).contains(p) = (a.contains(p.first) and b.contains(p.second))
    finite_set_product(a, b).contains(p) iff (a.contains(p.first) and b.contains(p.second))
}

/// Finite-product membership can be introduced from component membership.
theorem finite_set_product_contains_of_components[T, U](
    a: FiniteSet[T], b: FiniteSet[U], p: Pair[T, U]
) {
    a.contains(p.first) and b.contains(p.second) implies finite_set_product(a, b).contains(p)
} by {
    if a.contains(p.first) and b.contains(p.second) {
        finite_set_product_contains_pair(a, b, p.first, p.second)
        finite_set_product(a, b).contains(Pair.new(p.first, p.second))
        p = Pair.new(p.first, p.second)
        finite_set_product(a, b).contains(p)
    }
}

/// Finite-product membership projects to first-component membership.
theorem finite_set_product_contains_first[T, U](a: FiniteSet[T], b: FiniteSet[U], p: Pair[T, U]) {
    finite_set_product(a, b).contains(p) implies a.contains(p.first)
} by {
    if finite_set_product(a, b).contains(p) {
        finite_set_product_contains_left(a, b, p)
        a.contains(p.first)
    }
}

/// Finite-product membership projects to second-component membership.
theorem finite_set_product_contains_second[T, U](a: FiniteSet[T], b: FiniteSet[U], p: Pair[T, U]) {
    finite_set_product(a, b).contains(p) implies b.contains(p.second)
} by {
    if finite_set_product(a, b).contains(p) {
        finite_set_product_contains_right(a, b, p)
        b.contains(p.second)
    }
}

/// Finite-product subset is monotone in the left factor alone.
theorem finite_set_product_subset_left[T, U](a: FiniteSet[T], a_target: FiniteSet[T], b: FiniteSet[U]) {
    a.subset_eq(a_target) implies finite_set_product(a, b).subset_eq(finite_set_product(a_target, b))
} by {
    if a.subset_eq(a_target) {
        finite_set_product_subset(a, a_target, b, b)
        finite_set_product(a, b).subset_eq(finite_set_product(a_target, b))
    }
}

/// Finite-product subset is monotone in the right factor alone.
theorem finite_set_product_subset_right[T, U](a: FiniteSet[T], b: FiniteSet[U], b_target: FiniteSet[U]) {
    b.subset_eq(b_target) implies finite_set_product(a, b).subset_eq(finite_set_product(a, b_target))
} by {
    if b.subset_eq(b_target) {
        finite_set_product_subset(a, a, b, b_target)
        finite_set_product(a, b).subset_eq(finite_set_product(a, b_target))
    }
}

/// Fubini row sums, with the equality oriented from iterated rows to the flat product sum.
theorem finite_set_sum_product_fubini_rows_rev[T, U, A: AddCommMonoid](
    a: FiniteSet[T], b: FiniteSet[U], f: Pair[T, U] -> A
) {
    finite_set_sum(a, function(x: T) { finite_set_sum(b, function(y: U) { f(Pair.new(x, y)) }) }) =
        finite_set_sum(finite_set_product(a, b), f)
} by {
    finite_set_sum_product_fubini_rows(a, b, f)
    finite_set_sum(finite_set_product(a, b), f) =
        finite_set_sum(a, function(x: T) { finite_set_sum(b, function(y: U) { f(Pair.new(x, y)) }) })
}

/// The row function for a product summand evaluates by applying the summand to the pair.
theorem finite_set_product_row_fn_apply[T, U, A](f: Pair[T, U] -> A, x: T, y: U) {
    finite_set_product_row_fn(f, x, y) = f(Pair.new(x, y))
}

/// Real finite-set sums inherit nonnegativity from pointwise nonnegative summands.
theorem finite_set_sum_nonnegative_of_pointwise[T](s: FiniteSet[T], f: T -> Real) {
    (forall(item: T) { Real.0 <= f(item) }) implies Real.0 <= finite_set_sum(s, f)
} by {
    if forall(item: T) { Real.0 <= f(item) } {
        finite_set_sum_nonneg(s, f)
        Real.0 <= finite_set_sum(s, f)
    }
}

/// Real finite-set sums are monotone under pointwise order.
theorem finite_set_sum_monotone_of_pointwise_le[T](s: FiniteSet[T], f: T -> Real, g: T -> Real) {
    pointwise_le(f, g) implies finite_set_sum(s, f) <= finite_set_sum(s, g)
} by {
    if pointwise_le(f, g) {
        finite_set_sum_le(s, f, g)
        finite_set_sum(s, f) <= finite_set_sum(s, g)
    }
}

/// Pointwise order can be introduced from an explicit universal inequality.
theorem pointwise_le_intro[T](f: T -> Real, g: T -> Real) {
    (forall(t: T) { f(t) <= g(t) }) implies pointwise_le(f, g)
} by {
    if forall(t: T) { f(t) <= g(t) } {
        pointwise_le(f, g) = forall(t: T) { f(t) <= g(t) }
    }
}

/// Pointwise order is equivalent to its universal inequality expansion.
theorem pointwise_le_iff_forall[T](f: T -> Real, g: T -> Real) {
    pointwise_le(f, g) iff forall(t: T) { f(t) <= g(t) }
} by {
    if pointwise_le(f, g) {
        pointwise_le(f, g) = forall(t: T) { f(t) <= g(t) }
    }
    if forall(t: T) { f(t) <= g(t) } {
        pointwise_le_intro(f, g)
        pointwise_le(f, g)
    }
    pointwise_le(f, g) iff forall(t: T) { f(t) <= g(t) }
}

/// Instantiating a pointwise inequality, with theorem name matching monotonicity usage.
theorem pointwise_le_elim[T](f: T -> Real, g: T -> Real, t: T) {
    pointwise_le(f, g) implies f(t) <= g(t)
} by {
    if pointwise_le(f, g) {
        pointwise_le_at(f, g, t)
        f(t) <= g(t)
    }
}
