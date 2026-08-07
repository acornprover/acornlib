from list import List, map, map_length, map_contains, map_contains_of_contains,
    unique_is_smallest_containing_list, list_contains_implies_count_geq_one,
    unique_implies_no_duplicate, remove_elem_contains_iff,
    remove_element_in_unique_equals_length_minus_one
from nat import Nat
from nat import sum_lte
from data.basic.set import Set, subset_contains, set_ext, singleton_contains_eq,
    singleton_set_cardinality_is_one, list_set, list_set_contains_eq
from finite_set import FiniteSet, fs_from_list, fs_image, finite_set_has_unique_list,
    finite_set_maps_into_image, finite_set_image_contains_witness,
    finite_set_from_unique_list_cardinality_is_length, finite_set_cardinality_is_well_defined,
    finite_set_pigeonhole_into_finite_set
from data.basic.set_list_partition import set_list_union, set_list_union_contains_of_member,
    set_list_union_contains_witness, set_list_exactly_covers, set_list_pairwise_disjoint,
    set_list_partition_of, set_list_all_cardinality_is, set_list_all_cardinality_by,
    set_list_cardinality_sum, set_list_all_cardinality_is_map,
    set_list_all_cardinality_by_map, set_list_partition_cardinality_is,
    set_list_partition_cardinality_by

numerals Nat

lemma cons_unique_not_tail_contains_by_count[A](head: A, tail: List[A]) {
    List.cons[A](head, tail).is_unique implies not tail.contains(head)
} by {
    if tail.contains(head) {
        list_contains_implies_count_geq_one[A](tail, head)
        tail.count(head) >= Nat.1
        List.cons(head, tail).count(head) = Nat.1 + tail.count(head)
        sum_lte(Nat.1, Nat.1, Nat.1, tail.count(head))
        Nat.1 + Nat.1 <= Nat.1 + tail.count(head)
        Nat.1 + Nat.1 = Nat.1.suc
        Nat.1.suc <= Nat.1 + tail.count(head)
        List.cons(head, tail).count(head) >= Nat.1.suc
        List.cons(head, tail).count(head) > Nat.1
        unique_implies_no_duplicate[A](List.cons(head, tail), head)
        false
    }
}

lemma list_cons_unique_intro_generic[B](x: B, tail: List[B]) {
    not tail.contains(x) and tail.is_unique implies List.cons[B](x, tail).is_unique
} by {
}

/// A map that is injective on the members of a unique list preserves uniqueness.
theorem locally_injective_map_is_unique[T, U](items: List[T], f: T -> U) {
    items.is_unique and forall(x: T, y: T) {
        items.contains(x) and items.contains(y) and f(x) = f(y) implies x = y
    }
    implies map[T, U](items, f).is_unique
} by {
    define p(l: List[T]) -> Bool {
        l.is_unique and forall(x: T, y: T) {
            l.contains(x) and l.contains(y) and f(x) = f(y) implies x = y
        }
        implies map[T, U](l, f).is_unique
    }

    map[T, U](List.nil[T], f) = List.nil[U]
    List.nil[U].unique = List.nil[U]
    List.nil[U].is_unique
    p(List.nil[T])

    forall(head: T, tail: List[T]) {
        if p(tail) {
            if List.cons[T](head, tail).is_unique and forall(x: T, y: T) {
                List.cons[T](head, tail).contains(x) and List.cons[T](head, tail).contains(y) and f(x) = f(y) implies x = y
            } {
                tail.is_unique
                forall(x: T, y: T) {
                    if tail.contains(x) and tail.contains(y) and f(x) = f(y) {
                        List.cons[T](head, tail).contains(x)
                        List.cons[T](head, tail).contains(y)
                        x = y
                    }
                }
                tail.is_unique and forall(x: T, y: T) {
                    tail.contains(x) and tail.contains(y) and f(x) = f(y) implies x = y
                }
                if map[T, U](tail, f).contains(f(head)) {
                    map_contains[T, U](tail, f, f(head))
                    let x: T satisfy {
                        tail.contains(x) and f(x) = f(head)
                    }
                    List.cons[T](head, tail).contains(x)
                    List.cons[T](head, tail).contains(head)
                    x = head
                    cons_unique_not_tail_contains_by_count[T](head, tail)
                    false
                }
                list_cons_unique_intro_generic[U](f(head), map[T, U](tail, f))
                List.cons[U](f(head), map[T, U](tail, f)).is_unique
                map[T, U](List.cons[T](head, tail), f).is_unique
            }
            p(List.cons[T](head, tail))
        }
    }

    List.induction(function(l: List[T]) { p(l) })
    p(items)
}

/// A locally injective map on a unique list gives a unique image list of the same length.
theorem locally_injective_map_preserves_unique_length[T, U](items: List[T], f: T -> U) {
    items.is_unique and forall(x: T, y: T) {
        items.contains(x) and items.contains(y) and f(x) = f(y) implies x = y
    }
    implies map[T, U](items, f).is_unique and map[T, U](items, f).length = items.length
} by {
    if items.is_unique and forall(x: T, y: T) {
        items.contains(x) and items.contains(y) and f(x) = f(y) implies x = y
    } {
        locally_injective_map_is_unique[T, U](items, f)
        map_length[T, U](items, f)
        map[T, U](items, f).length = items.length
    }
}

/// A map from a unique list into a shorter list identifies two distinct members of the source.
theorem finite_list_pigeonhole_into_shorter_list[T, U](items: List[T], targets: List[U], f: T -> U) {
    items.is_unique and items.length > targets.length and
    forall(x: T) {
        items.contains(x) implies targets.contains(f(x))
    } implies exists(x: T, y: T) {
        items.contains(x) and items.contains(y) and x != y and f(x) = f(y)
    }
} by {
    if items.is_unique and items.length > targets.length and forall(x: T) {
        items.contains(x) implies targets.contains(f(x))
    } {
        if not exists(x: T, y: T) {
            items.contains(x) and items.contains(y) and x != y and f(x) = f(y)
        } {
            forall(x: T, y: T) {
                if items.contains(x) and items.contains(y) and f(x) = f(y) {
                    if x != y {
                        false
                    }
                    x = y
                }
            }
            locally_injective_map_is_unique(items, f)
            map[T, U](items, f).is_unique
            map[T, U](items, f).unique = map[T, U](items, f)
            map_length[T, U](items, f)
            map[T, U](items, f).unique.length = items.length
            forall(value: U) {
                if map[T, U](items, f).contains(value) {
                    map_contains[T, U](items, f, value)
                    let source_value: T satisfy {
                        items.contains(source_value) and f(source_value) = value
                    }
                    targets.contains(value)
                }
            }
            unique_is_smallest_containing_list(map[T, U](items, f), targets)
            map[T, U](items, f).unique.length <= targets.length
            items.length <= targets.length
            not items.length > targets.length
            false
        }
    }
}

/// A locally injective map into an equal-length unique list covers every target member.
theorem finite_list_local_injection_is_surjective[T, U](items: List[T], targets: List[U], f: T -> U) {
    items.is_unique and targets.is_unique and items.length = targets.length and
    forall(x: T, y: T) {
        items.contains(x) and items.contains(y) and f(x) = f(y) implies x = y
    } and forall(x: T) {
        items.contains(x) implies targets.contains(f(x))
    } implies forall(value: U) {
        targets.contains(value) implies exists(x: T) {
            items.contains(x) and f(x) = value
        }
    }
} by {
    if items.is_unique and targets.is_unique and items.length = targets.length and
        forall(x: T, y: T) {
            items.contains(x) and items.contains(y) and f(x) = f(y) implies x = y
        } and forall(x: T) {
            items.contains(x) implies targets.contains(f(x))
        } {
        forall(value: U) {
            if targets.contains(value) {
                if not exists(x: T) {
                    items.contains(x) and f(x) = value
                } {
                    forall(x: T) {
                        if items.contains(x) {
                            targets.contains(f(x))
                            if f(x) = value {
                                false
                            }
                            f(x) != value
                            remove_elem_contains_iff(targets, value, f(x))
                            targets.remove_elem(value).contains(f(x))
                        }
                    }
                    targets.unique = targets
                    remove_element_in_unique_equals_length_minus_one(targets, value)
                    targets.remove_elem(value).length + Nat.1 = targets.length
                    targets.remove_elem(value).length + Nat.1 = items.length
                    targets.remove_elem(value).length + Nat.1 > targets.remove_elem(value).length
                    items.length > targets.remove_elem(value).length
                    targets.remove_elem(value).length < items.length
                    finite_list_pigeonhole_into_shorter_list(items, targets.remove_elem(value), f)
                    let (x: T, y: T) satisfy {
                        items.contains(x) and items.contains(y) and x != y and f(x) = f(y)
                    }
                    x = y
                    false
                }
            }
        }
    }
}

/// The characteristic predicate of a fiber of a finite source set over a map value.
define finite_fiber_contains[T, U](source: FiniteSet[T], f: T -> U, value: U, x: T) -> Bool {
    source.contains(x) and f(x) = value
}

/// The fiber of a finite source set over a value of a map.
define finite_fiber_set[T, U](source: FiniteSet[T], f: T -> U, value: U) -> Set[T] {
    Set[T].new(finite_fiber_contains(source, f, value))
}

/// The list of fibers indexed by a list of target values.
define finite_fiber_set_list[T, U](source: FiniteSet[T], f: T -> U, values: List[U]) -> List[Set[T]] {
    map[U, Set[T]](values, function(value: U) { finite_fiber_set(source, f, value) })
}

/// Membership in a finite fiber is source membership together with the prescribed map value.
theorem finite_fiber_set_contains_eq[T, U](source: FiniteSet[T], f: T -> U, value: U, x: T) {
    finite_fiber_set(source, f, value).contains(x) = (source.contains(x) and f(x) = value)
}

/// A source element lies in the fiber over its image.
theorem finite_fiber_set_contains_of_source[T, U](source: FiniteSet[T], f: T -> U, x: T) {
    source.contains(x) implies finite_fiber_set(source, f, f(x)).contains(x)
} by {
    if source.contains(x) {
        finite_fiber_set_contains_eq(source, f, f(x), x)
        finite_fiber_set(source, f, f(x)).contains(x)
    }
}

/// If the index values contain every image of a source element, the source is contained in the fiber union.
theorem finite_fiber_set_list_source_subset_union[T, U](source: FiniteSet[T], f: T -> U, values: List[U]) {
    (forall(x: T) { source.contains(x) implies values.contains(f(x)) }) implies
    source.underlying_set.subset(set_list_union[T](finite_fiber_set_list(source, f, values)))
} by {
    let family = function(value: U) { finite_fiber_set(source, f, value) }
    let fibers = finite_fiber_set_list(source, f, values)
    if forall(x: T) { source.contains(x) implies values.contains(f(x)) } {
        forall(x: T) {
            if source.underlying_set.contains(x) {
                source.contains(x)
                values.contains(f(x))
                finite_fiber_set_contains_of_source(source, f, x)
                map_contains_of_contains[U, Set[T]](values, family, f(x))
                fibers.contains(finite_fiber_set(source, f, f(x)))
                set_list_union_contains_of_member[T](fibers, finite_fiber_set(source, f, f(x)), x)
                set_list_union[T](fibers).contains(x)
            }
        }
        source.underlying_set.subset(set_list_union[T](finite_fiber_set_list(source, f, values)))
    }
}

/// If the index values contain every image of a source element, their fiber list exactly covers the source.
theorem finite_fiber_set_list_exactly_covers_source[T, U](source: FiniteSet[T], f: T -> U, values: List[U]) {
    (forall(x: T) { source.contains(x) implies values.contains(f(x)) }) implies
    set_list_exactly_covers[T](finite_fiber_set_list(source, f, values), source.underlying_set)
} by {
    let fibers = finite_fiber_set_list(source, f, values)
    let whole = source.underlying_set
    if forall(x: T) { source.contains(x) implies values.contains(f(x)) } {
        finite_fiber_set_list_source_subset_union(source, f, values)
        forall(x: T) {
            if set_list_union[T](fibers).contains(x) {
                set_list_union_contains_witness[T](fibers, x)
                let c: Set[T] satisfy {
                    fibers.contains(c) and c.contains(x)
                }
                map_contains[U, Set[T]](values, function(value: U) { finite_fiber_set(source, f, value) }, c)
                let value: U satisfy {
                    values.contains(value) and finite_fiber_set(source, f, value) = c
                }
                finite_fiber_set_contains_eq(source, f, value, x)
                source.contains(x)
                whole.contains(x)
            }
            if whole.contains(x) {
                finite_fiber_set_list_source_subset_union(source, f, values)
                source.underlying_set.subset(set_list_union[T](finite_fiber_set_list(source, f, values)))
                subset_contains(source.underlying_set, set_list_union[T](finite_fiber_set_list(source, f, values)), x)
                set_list_union[T](fibers).contains(x)
            }
            set_list_union[T](fibers).contains(x) = whole.contains(x)
        }
        set_ext(set_list_union[T](fibers), whole)
        set_list_exactly_covers[T](finite_fiber_set_list(source, f, values), source.underlying_set)
    }
}

/// Fibers over a list of values are pairwise disjoint as sets.
theorem finite_fiber_set_list_pairwise_disjoint[T, U](source: FiniteSet[T], f: T -> U, values: List[U]) {
    set_list_pairwise_disjoint[T](finite_fiber_set_list(source, f, values))
} by {
    let family = function(value: U) { finite_fiber_set(source, f, value) }
    let fibers = finite_fiber_set_list(source, f, values)
    forall(a: Set[T], b: Set[T]) {
        if fibers.contains(a) and fibers.contains(b) and a != b {
            map_contains[U, Set[T]](values, family, a)
            let va: U satisfy {
                values.contains(va) and family(va) = a
            }
            map_contains[U, Set[T]](values, family, b)
            let vb: U satisfy {
                values.contains(vb) and family(vb) = b
            }
            forall(x: T) {
                if a.contains(x) and b.contains(x) {
                    finite_fiber_set_contains_eq(source, f, va, x)
                    f(x) = va
                    finite_fiber_set_contains_eq(source, f, vb, x)
                    f(x) = vb
                    false
                }
            }
            a.is_disjoint(b)
        }
    }
    forall(a: Set[T], b: Set[T]) {
        fibers.contains(a) and fibers.contains(b) and a != b implies a.is_disjoint(b)
    }
    set_list_pairwise_disjoint[T](fibers)
}

/// A chosen unique list of the finite image indexes the canonical fiber partition.
let finite_fiber_partition_values[T, U](source: FiniteSet[T], f: T -> U) -> result: List[U] satisfy {
    fs_from_list(result) = fs_image(source, f) and result.is_unique
} by {
    finite_set_has_unique_list(fs_image(source, f))
}

/// The chosen fiber-index list represents the finite image and has no duplicates.
theorem finite_fiber_partition_values_spec[T, U](source: FiniteSet[T], f: T -> U) {
    fs_from_list(finite_fiber_partition_values(source, f)) = fs_image(source, f) and
    finite_fiber_partition_values(source, f).is_unique
}

/// The canonical finite fiber partition over the image of a finite source set.
define finite_fiber_partition[T, U](source: FiniteSet[T], f: T -> U) -> List[Set[T]] {
    finite_fiber_set_list(source, f, finite_fiber_partition_values(source, f))
}

/// The image of any source element is present in the canonical fiber-index list.
theorem finite_fiber_partition_values_contains_image[T, U](source: FiniteSet[T], f: T -> U, x: T) {
    source.contains(x) implies finite_fiber_partition_values(source, f).contains(f(x))
} by {
    if source.contains(x) {
        let values = finite_fiber_partition_values(source, f)
        finite_fiber_partition_values_spec(source, f)
        finite_set_maps_into_image(source, f, x)
        fs_from_list(values).contains(f(x))
        fs_from_list(values).underlying_set = list_set(values)
        list_set(values).contains(f(x))
        list_set_contains_eq(values, f(x))
        finite_fiber_partition_values(source, f).contains(f(x))
    }
}

/// Any canonical fiber index is the image of some source element.
theorem finite_fiber_partition_values_has_preimage[T, U](source: FiniteSet[T], f: T -> U, value: U) {
    finite_fiber_partition_values(source, f).contains(value) implies exists(x: T) {
        source.contains(x) and value = f(x)
    }
} by {
    if finite_fiber_partition_values(source, f).contains(value) {
        let values = finite_fiber_partition_values(source, f)
        finite_fiber_partition_values_spec(source, f)
        fs_from_list(values).underlying_set = list_set(values)
        list_set_contains_eq(values, value)
        finite_set_image_contains_witness(source, f, value)
        exists(x: T) {
            source.contains(x) and value = f(x)
        }
    }
}

/// The canonical finite fiber partition has no duplicate fiber sets.
theorem finite_fiber_partition_is_unique[T, U](source: FiniteSet[T], f: T -> U) {
    finite_fiber_partition(source, f).is_unique
} by {
    let values = finite_fiber_partition_values(source, f)
    let family = function(value: U) { finite_fiber_set(source, f, value) }
    finite_fiber_partition_values_spec(source, f)
    values.is_unique
    forall(a: U, b: U) {
        if values.contains(a) and values.contains(b) and family(a) = family(b) {
            finite_fiber_partition_values_has_preimage(source, f, a)
            let x: T satisfy {
                source.contains(x) and a = f(x)
            }
            finite_fiber_set_contains_eq(source, f, a, x)
            finite_fiber_set_contains_eq(source, f, b, x)
            f(x) = b
            a = b
        }
    }
    forall(a: U, b: U) {
        values.contains(a) and values.contains(b) and family(a) = family(b) implies a = b
    }
    locally_injective_map_is_unique[U, Set[T]](values, family)
    map[U, Set[T]](values, family).is_unique
    finite_fiber_set_list(source, f, values) = map[U, Set[T]](values, family)
}

/// The canonical finite fiber partition is pairwise disjoint.
theorem finite_fiber_partition_pairwise_disjoint[T, U](source: FiniteSet[T], f: T -> U) {
    set_list_pairwise_disjoint[T](finite_fiber_partition(source, f))
} by {
    let values = finite_fiber_partition_values(source, f)
    finite_fiber_set_list_pairwise_disjoint(source, f, values)
}

/// The canonical finite fiber partition exactly covers the finite source set.
theorem finite_fiber_partition_exactly_covers_source[T, U](source: FiniteSet[T], f: T -> U) {
    set_list_exactly_covers[T](finite_fiber_partition(source, f), source.underlying_set)
} by {
    let values = finite_fiber_partition_values(source, f)
    forall(x: T) {
        if source.contains(x) {
            finite_fiber_partition_values_contains_image(source, f, x)
            values.contains(f(x))
        }
    }
    finite_fiber_set_list_exactly_covers_source(source, f, values)
    set_list_exactly_covers[T](finite_fiber_set_list(source, f, values), source.underlying_set)
}

/// The canonical finite fiber partition is an exact unique pairwise-disjoint cover of the source.
theorem finite_fiber_partition_is_partition[T, U](source: FiniteSet[T], f: T -> U) {
    set_list_partition_of[T](finite_fiber_partition(source, f), source.underlying_set)
} by {
    finite_fiber_partition_exactly_covers_source(source, f)
    finite_fiber_partition_is_unique(source, f)
    finite_fiber_partition_pairwise_disjoint(source, f)
}

/// The source cardinality is `n` times the number of canonical fibers when every fiber has cardinality `n`.
theorem finite_fiber_partition_cardinality_is[T, U](source: FiniteSet[T], f: T -> U, n: Nat) {
    set_list_all_cardinality_is[T](finite_fiber_partition(source, f), n)
    implies source.underlying_set.cardinality_is(n * finite_fiber_partition(source, f).length)
} by {
    if set_list_all_cardinality_is[T](finite_fiber_partition(source, f), n) {
        finite_fiber_partition_is_partition(source, f)
        set_list_partition_cardinality_is[T](finite_fiber_partition(source, f), source.underlying_set, n)
        source.underlying_set.cardinality_is(n * finite_fiber_partition(source, f).length)
    }
}

/// Pointwise equal cardinalities for image-indexed fibers give source cardinality `n` times the image-list length.
theorem finite_fiber_partition_cardinality_is_values[T, U](source: FiniteSet[T], f: T -> U,
    n: Nat) {
    (forall(value: U) {
        finite_fiber_partition_values(source, f).contains(value) implies
        finite_fiber_set(source, f, value).cardinality_is(n)
    }) implies source.underlying_set.cardinality_is(n * finite_fiber_partition_values(source, f).length)
} by {
    let values = finite_fiber_partition_values(source, f)
    let family = function(value: U) { finite_fiber_set(source, f, value) }
    if forall(value: U) {
        finite_fiber_partition_values(source, f).contains(value) implies
        finite_fiber_set(source, f, value).cardinality_is(n)
    } {
        forall(value: U) {
            if values.contains(value) {
                family(value).cardinality_is(n)
            }
        }
        set_list_all_cardinality_is_map[T, U](values, family, n)
        set_list_all_cardinality_is[T](map[U, Set[T]](values, family), n)
        finite_fiber_set_list(source, f, values) = map[U, Set[T]](values, family)
        map_length[U, Set[T]](values, family)
        finite_fiber_partition_cardinality_is(source, f, n)
        source.underlying_set.cardinality_is(n * finite_fiber_partition(source, f).length)
        source.underlying_set.cardinality_is(n * finite_fiber_partition_values(source, f).length)
    }
}

/// The canonical image-index list has length equal to the cardinality of the finite image.
theorem finite_fiber_partition_image_cardinality_is_values_length[T, U](source: FiniteSet[T],
    f: T -> U) {
    fs_image(source, f).cardinality_is(finite_fiber_partition_values(source, f).length)
} by {
    let values = finite_fiber_partition_values(source, f)
    finite_fiber_partition_values_spec(source, f)
    finite_set_from_unique_list_cardinality_is_length[U](values)
}

/// Constant canonical fiber cardinality and image cardinality determine the source cardinality.
theorem finite_fiber_partition_constant_fiber_cardinality_by_image[T, U](source: FiniteSet[T],
    f: T -> U, n: Nat, m: Nat) {
    (forall(value: U) {
        finite_fiber_partition_values(source, f).contains(value) implies
        finite_fiber_set(source, f, value).cardinality_is(n)
    }) and fs_image(source, f).cardinality_is(m)
    implies source.underlying_set.cardinality_is(n * m)
} by {
    if forall(value: U) {
        finite_fiber_partition_values(source, f).contains(value) implies
        finite_fiber_set(source, f, value).cardinality_is(n)
    } and fs_image(source, f).cardinality_is(m) {
        let values = finite_fiber_partition_values(source, f)
        finite_fiber_partition_cardinality_is_values(source, f, n)
        finite_fiber_partition_image_cardinality_is_values_length(source, f)
        finite_set_cardinality_is_well_defined[U](fs_image(source, f), m, values.length)
        n * m = n * values.length
        source.underlying_set.cardinality_is(n * m)
    }
}

/// Every canonical image value indexes a nonempty fiber.
theorem finite_fiber_partition_value_fiber_nonempty[T, U](source: FiniteSet[T], f: T -> U,
    value: U) {
    finite_fiber_partition_values(source, f).contains(value) implies exists(x: T) {
        finite_fiber_set(source, f, value).contains(x)
    }
} by {
    if finite_fiber_partition_values(source, f).contains(value) {
        finite_fiber_partition_values_has_preimage(source, f, value)
        let x: T satisfy {
            source.contains(x) and value = f(x)
        }
        finite_fiber_set_contains_eq(source, f, value, x)
        exists(w: T) {
            w = x and finite_fiber_set(source, f, value).contains(w)
        }
    }
}

/// Canonical image-list membership is exactly existence of a source preimage.
theorem finite_fiber_partition_values_contains_eq_preimage[T, U](source: FiniteSet[T],
    f: T -> U, value: U) {
    finite_fiber_partition_values(source, f).contains(value) = exists(x: T) {
        source.contains(x) and value = f(x)
    }
} by {
    if finite_fiber_partition_values(source, f).contains(value) {
        finite_fiber_partition_values_has_preimage(source, f, value)
        exists(x: T) {
            source.contains(x) and value = f(x)
        }
    }
    if exists(x: T) {
        source.contains(x) and value = f(x)
    } {
        let x: T satisfy {
            source.contains(x) and value = f(x)
        }
        finite_fiber_partition_values_contains_image(source, f, x)
        finite_fiber_partition_values(source, f).contains(value)
    }
}

/// A canonical image value is present exactly when its canonical fiber is nonempty.
theorem finite_fiber_partition_value_fiber_nonempty_eq_contains[T, U](source: FiniteSet[T],
    f: T -> U, value: U) {
    finite_fiber_partition_values(source, f).contains(value) = exists(x: T) {
        finite_fiber_set(source, f, value).contains(x)
    }
} by {
    if finite_fiber_partition_values(source, f).contains(value) {
        finite_fiber_partition_value_fiber_nonempty(source, f, value)
        exists(x: T) {
            finite_fiber_set(source, f, value).contains(x)
        }
    }
    if exists(x: T) {
        finite_fiber_set(source, f, value).contains(x)
    } {
        let x: T satisfy {
            finite_fiber_set(source, f, value).contains(x)
        }
        finite_fiber_set_contains_eq(source, f, value, x)
        finite_fiber_partition_values_contains_image(source, f, x)
        finite_fiber_partition_values(source, f).contains(value)
    }
}

/// For a source-injective map, the fiber over a source value is the singleton containing that source element.
theorem finite_fiber_partition_injective_fiber_eq_singleton_of_preimage[T, U](source: FiniteSet[T],
    f: T -> U, value: U, x: T) {
    (forall(a: T, b: T) {
        source.contains(a) and source.contains(b) and f(a) = f(b) implies a = b
    }) and source.contains(x) and value = f(x) implies
    finite_fiber_set(source, f, value) = Set[T].singleton(x)
} by {
    if forall(a: T, b: T) {
        source.contains(a) and source.contains(b) and f(a) = f(b) implies a = b
    } and source.contains(x) and value = f(x) {
        let fiber = finite_fiber_set(source, f, value)
        forall(y: T) {
            if fiber.contains(y) {
                finite_fiber_set_contains_eq(source, f, value, y)
                f(y) = value
                f(y) = f(x)
                source.contains(y) and source.contains(x) and f(y) = f(x)
                singleton_contains_eq[T](x, y)
                Set[T].singleton(x).contains(y)
            }
            if Set[T].singleton(x).contains(y) {
                singleton_contains_eq[T](x, y)
                finite_fiber_set_contains_eq(source, f, value, y)
                fiber.contains(y)
            }
            fiber.contains(y) = Set[T].singleton(x).contains(y)
        }
        set_ext[T](fiber, Set[T].singleton(x))
        finite_fiber_set(source, f, value) = Set[T].singleton(x)
    }
}

/// A source-injective map makes every canonical fiber an actual singleton.
theorem finite_fiber_partition_injective_value_fiber_is_singleton[T, U](source: FiniteSet[T],
    f: T -> U, value: U) {
    (forall(a: T, b: T) {
        source.contains(a) and source.contains(b) and f(a) = f(b) implies a = b
    }) and finite_fiber_partition_values(source, f).contains(value) implies exists(x: T) {
        source.contains(x) and value = f(x) and finite_fiber_set(source, f, value) = Set[T].singleton(x)
    }
} by {
    if forall(a: T, b: T) {
        source.contains(a) and source.contains(b) and f(a) = f(b) implies a = b
    } and finite_fiber_partition_values(source, f).contains(value) {
        finite_fiber_partition_values_has_preimage(source, f, value)
        let x: T satisfy {
            source.contains(x) and value = f(x)
        }
        finite_fiber_partition_injective_fiber_eq_singleton_of_preimage(source, f, value, x)
        exists(w: T) {
            source.contains(w) and value = f(w) and finite_fiber_set(source, f, value) = Set[T].singleton(w)
        }
    }
}

/// A source-injective map has singleton canonical fibers over every image value.
theorem finite_fiber_partition_injective_fibers_cardinality_one[T, U](source: FiniteSet[T],
    f: T -> U) {
    (forall(x: T, y: T) {
        source.contains(x) and source.contains(y) and f(x) = f(y) implies x = y
    }) implies forall(value: U) {
        finite_fiber_partition_values(source, f).contains(value) implies
        finite_fiber_set(source, f, value).cardinality_is(Nat.1)
    }
} by {
    if forall(x: T, y: T) {
        source.contains(x) and source.contains(y) and f(x) = f(y) implies x = y
    } {
        forall(value: U) {
            if finite_fiber_partition_values(source, f).contains(value) {
                finite_fiber_partition_values_has_preimage(source, f, value)
                let x: T satisfy {
                    source.contains(x) and value = f(x)
                }
                let fiber = finite_fiber_set(source, f, value)
                forall(y: T) {
                    if fiber.contains(y) {
                        finite_fiber_set_contains_eq(source, f, value, y)
                        f(y) = value
                        f(y) = f(x)
                        source.contains(y) and source.contains(x) and f(y) = f(x)
                        singleton_contains_eq[T](x, y)
                        Set[T].singleton(x).contains(y)
                    }
                    if Set[T].singleton(x).contains(y) {
                        singleton_contains_eq[T](x, y)
                        finite_fiber_set_contains_eq(source, f, value, y)
                        fiber.contains(y)
                    }
                    fiber.contains(y) = Set[T].singleton(x).contains(y)
                }
                set_ext[T](fiber, Set[T].singleton(x))
                singleton_set_cardinality_is_one[T](x)
                finite_fiber_set(source, f, value).cardinality_is(Nat.1)
            }
            if finite_fiber_partition_values(source, f).contains(value) {
                finite_fiber_set(source, f, value).cardinality_is(Nat.1)
            }
        }
        forall(value: U) {
            if finite_fiber_partition_values(source, f).contains(value) {
                finite_fiber_set(source, f, value).cardinality_is(Nat.1)
            }
        }
    }
}


/// Singleton canonical fibers make the source cardinality equal to the image cardinality.
theorem finite_fiber_partition_singleton_fibers_cardinality_eq_image[T, U](source: FiniteSet[T],
    f: T -> U, m: Nat) {
    (forall(value: U) {
        finite_fiber_partition_values(source, f).contains(value) implies
        finite_fiber_set(source, f, value).cardinality_is(Nat.1)
    }) and fs_image(source, f).cardinality_is(m) implies source.cardinality_is(m)
} by {
    if forall(value: U) {
        finite_fiber_partition_values(source, f).contains(value) implies
        finite_fiber_set(source, f, value).cardinality_is(Nat.1)
    } and fs_image(source, f).cardinality_is(m) {
        finite_fiber_partition_constant_fiber_cardinality_by_image(source, f, Nat.1, m)
        source.cardinality_is(m)
    }
}

/// If a finite source has strictly larger cardinality than its image, the map has a collision.
theorem finite_fiber_partition_image_cardinality_pigeonhole_collision[T, U](source: FiniteSet[T],
    f: T -> U, n_source: Nat, n_image: Nat) {
    source.cardinality_is(n_source) and fs_image(source, f).cardinality_is(n_image) and
    n_image < n_source implies exists(x: T, y: T) {
        x != y and f(x) = f(y)
    }
} by {
    if source.cardinality_is(n_source) and fs_image(source, f).cardinality_is(n_image) and
        n_image < n_source {
        forall(x: T) {
            if source.contains(x) {
                finite_set_maps_into_image(source, f, x)
                fs_image(source, f).contains(f(x))
            }
        }
        finite_set_pigeonhole_into_finite_set(source, fs_image(source, f), f, n_source, n_image)
    }
}

/// If the canonical image-index list is shorter than the finite source, the map has a collision.
theorem finite_fiber_partition_values_length_pigeonhole_collision[T, U](source: FiniteSet[T],
    f: T -> U, n_source: Nat) {
    source.cardinality_is(n_source) and finite_fiber_partition_values(source, f).length < n_source
    implies exists(x: T, y: T) {
        x != y and f(x) = f(y)
    }
} by {
    if source.cardinality_is(n_source) and finite_fiber_partition_values(source, f).length < n_source {
        let n_image = finite_fiber_partition_values(source, f).length
        finite_fiber_partition_image_cardinality_is_values_length(source, f)
        fs_image(source, f).cardinality_is(n_image)
        finite_fiber_partition_image_cardinality_pigeonhole_collision(source, f, n_source, n_image)
        exists(x: T, y: T) {
            x != y and f(x) = f(y)
        }
    }
}

/// The source cardinality is the sum of cardinalities over the canonical finite fibers.
theorem finite_fiber_partition_cardinality_by[T, U](source: FiniteSet[T], f: T -> U,
    card: Set[T] -> Nat) {
    set_list_all_cardinality_by[T](finite_fiber_partition(source, f), card)
    implies source.underlying_set.cardinality_is(set_list_cardinality_sum[T](finite_fiber_partition(source, f), card))
} by {
    if set_list_all_cardinality_by[T](finite_fiber_partition(source, f), card) {
        finite_fiber_partition_is_partition(source, f)
        set_list_partition_cardinality_by[T](finite_fiber_partition(source, f), source.underlying_set, card)
        source.underlying_set.cardinality_is(set_list_cardinality_sum[T](finite_fiber_partition(source, f), card))
    }
}

/// Pointwise cardinalities for each image-indexed fiber give the source cardinality as a fiber sum.
theorem finite_fiber_partition_cardinality_by_values[T, U](source: FiniteSet[T], f: T -> U,
    card: Set[T] -> Nat) {
    (forall(value: U) {
        finite_fiber_partition_values(source, f).contains(value) implies
        finite_fiber_set(source, f, value).cardinality_is(card(finite_fiber_set(source, f, value)))
    }) implies source.underlying_set.cardinality_is(
        set_list_cardinality_sum[T](finite_fiber_partition(source, f), card))
} by {
    let values = finite_fiber_partition_values(source, f)
    let family = function(value: U) { finite_fiber_set(source, f, value) }
    if forall(value: U) {
        finite_fiber_partition_values(source, f).contains(value) implies
        finite_fiber_set(source, f, value).cardinality_is(card(finite_fiber_set(source, f, value)))
    } {
        forall(value: U) {
            if values.contains(value) {
                finite_fiber_set(source, f, value).cardinality_is(card(finite_fiber_set(source, f, value)))
                family(value).cardinality_is(card(family(value)))
            }
        }
        set_list_all_cardinality_by_map[T, U](values, family, card)
        set_list_all_cardinality_by[T](map[U, Set[T]](values, family), card)
        finite_fiber_set_list(source, f, values) = map[U, Set[T]](values, family)
        finite_fiber_partition_cardinality_by(source, f, card)
        source.underlying_set.cardinality_is(set_list_cardinality_sum[T](finite_fiber_partition(source, f), card))
    }
}
