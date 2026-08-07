from nat import Nat
from list import List
from finite_set import FiniteSet, fs_from_list, fs_insert, finite_set_ext_contains
from data.finite.finite_set_membership import fs_from_list_contains_eq, fs_insert_contains_eq
from data.list.list_cons_membership import cons_contains_eq

numerals Nat

/// The finite set of the empty list is empty.
theorem fs_from_list_nil[T] {
    fs_from_list(List.nil[T]) = FiniteSet.empty[T]
} by {
    forall(x: T) {
        fs_from_list_contains_eq(List.nil[T], x)
        fs_from_list(List.nil[T]).contains(x) = List.nil[T].contains(x)
        not List.nil[T].contains(x)
        not fs_from_list(List.nil[T]).contains(x)
        not FiniteSet.empty[T].contains(x)
        fs_from_list(List.nil[T]).contains(x) = FiniteSet.empty[T].contains(x)
        fs_from_list(List.nil[T]).contains(x) = fs_from_list(List.nil[T]).underlying_set.contains(x)
        FiniteSet.empty[T].contains(x) = FiniteSet.empty[T].underlying_set.contains(x)
        fs_from_list(List.nil[T]).underlying_set.contains(x) = FiniteSet.empty[T].underlying_set.contains(x)
    }
    finite_set_ext_contains(fs_from_list(List.nil[T]), FiniteSet.empty[T])
    fs_from_list(List.nil[T]) = FiniteSet.empty[T]
}

/// Prepending to a list inserts into its finite set.
///
/// This is the step that turns list induction into finite-set induction.
theorem fs_from_list_cons[T](head: T, tail: List[T]) {
    fs_from_list(List.cons(head, tail)) = fs_insert(fs_from_list(tail), head)
} by {
    forall(x: T) {
        fs_from_list_contains_eq(List.cons(head, tail), x)
        fs_from_list(List.cons(head, tail)).contains(x) = List.cons(head, tail).contains(x)
        cons_contains_eq(head, tail, x)
        List.cons(head, tail).contains(x) = (head = x or tail.contains(x))
        fs_from_list_contains_eq(tail, x)
        fs_from_list(tail).contains(x) = tail.contains(x)
        fs_insert_contains_eq(fs_from_list(tail), head, x)
        fs_insert(fs_from_list(tail), head).contains(x) = (x = head or fs_from_list(tail).contains(x))
        fs_from_list(List.cons(head, tail)).contains(x) = fs_insert(fs_from_list(tail), head).contains(x)
        fs_from_list(List.cons(head, tail)).contains(x) = fs_from_list(List.cons(head, tail)).underlying_set.contains(x)
        fs_insert(fs_from_list(tail), head).contains(x) = fs_insert(fs_from_list(tail), head).underlying_set.contains(x)
        fs_from_list(List.cons(head, tail)).underlying_set.contains(x) = fs_insert(fs_from_list(tail), head).underlying_set.contains(x)
    }
    finite_set_ext_contains(fs_from_list(List.cons(head, tail)), fs_insert(fs_from_list(tail), head))
    fs_from_list(List.cons(head, tail)) = fs_insert(fs_from_list(tail), head)
}
