from nat import Nat
from list import List
from finite_set import FiniteSet, fs_image, finite_set_image_cardinality_at_most,
    finite_set_cardinality_is_smallest_cardinality
from data.finite.finite_set_card import fs_card, fs_card_cardinality_is
from data.finite.finite_set_card_compare import fs_card_le_of_at_most

numerals Nat

/// A finite set is bounded by its own cardinality.
///
/// The bridge from the exact count to the bound predicate, which is the form the image lemma
/// takes its hypothesis in.
theorem fs_card_at_most_self[T](s: FiniteSet[T]) {
    s.cardinality_at_most(fs_card(s))
} by {
    fs_card_cardinality_is(s)
    s.cardinality_is(fs_card(s))
    finite_set_cardinality_is_smallest_cardinality(s, fs_card(s))
    s.underlying_set.cardinality_at_most(fs_card(s))
    s.cardinality_at_most(fs_card(s))
}

/// The image of a finite set is no larger than the set.
///
/// A map can identify members but never create new ones, so counting the image can only go
/// down. `src/finite_set/` states this as a bound on `cardinality_at_most`; this is the form
/// in terms of the numeric cardinality, which is what counting arguments use.
theorem fs_card_image_le[T, U](s: FiniteSet[T], f: T -> U) {
    fs_card(fs_image(s, f)) <= fs_card(s)
} by {
    fs_card_at_most_self(s)
    s.cardinality_at_most(fs_card(s))
    finite_set_image_cardinality_at_most(s, f, fs_card(s))
    fs_image(s, f).cardinality_at_most(fs_card(s))
    fs_card_le_of_at_most(fs_image(s, f), fs_card(s))
    fs_card(fs_image(s, f)) <= fs_card(s)
}
