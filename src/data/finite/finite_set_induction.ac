from nat import Nat
from list import List
from finite_set import FiniteSet, fs_from_list, fs_insert,
    finite_set_cardinality_has_exact_unique_list
from data.finite.finite_set_card import fs_card, fs_card_cardinality_is

numerals Nat

/// Every finite set is the set of members of some list without repeats.
///
/// `finite_set_sum` is already defined through such a representation, but the
/// representation itself was never exposed. It is what lets a statement about finite
/// sets be reduced to a statement about lists, where induction is available.
theorem finite_set_has_unique_list[T](s: FiniteSet[T]) {
    exists(items: List[T]) {
        fs_from_list(items) = s and items.is_unique
    }
} by {
    fs_card_cardinality_is(s)
    s.cardinality_is(fs_card(s))
    finite_set_cardinality_has_exact_unique_list(s, fs_card(s))
    exists(items: List[T]) {
        fs_from_list(items) = s and items.is_unique and items.length = fs_card(s)
    }
    let (rep: List[T]) satisfy {
        fs_from_list(rep) = s and rep.is_unique and rep.length = fs_card(s)
    }
    fs_from_list(rep) = s
    rep.is_unique
    exists(other: List[T]) {
        fs_from_list(other) = s and other.is_unique
    }
}

/// A finite set has a unique-list representation of its own size.
theorem finite_set_has_unique_list_of_card[T](s: FiniteSet[T]) {
    exists(items: List[T]) {
        fs_from_list(items) = s and items.is_unique and items.length = fs_card(s)
    }
} by {
    fs_card_cardinality_is(s)
    s.cardinality_is(fs_card(s))
    finite_set_cardinality_has_exact_unique_list(s, fs_card(s))
}

/// A property that holds of every unique-list representation holds of every finite set.
///
/// This is the transfer principle: prove something for lists without repeats, and it
/// follows for all finite sets. Induction on the list is then available.
theorem finite_set_of_list_property[T](p: FiniteSet[T] -> Bool) {
    (forall(items: List[T]) { items.is_unique implies p(fs_from_list(items)) })
        implies forall(s: FiniteSet[T]) { p(s) }
} by {
    if forall(items: List[T]) { items.is_unique implies p(fs_from_list(items)) } {
        forall(s: FiniteSet[T]) {
            finite_set_has_unique_list(s)
            let (rep: List[T]) satisfy {
                fs_from_list(rep) = s and rep.is_unique
            }
            rep.is_unique
            rep.is_unique implies p(fs_from_list(rep))
            p(fs_from_list(rep))
            fs_from_list(rep) = s
            p(s)
        }
    }
}
