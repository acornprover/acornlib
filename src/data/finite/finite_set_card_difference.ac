from nat import Nat
from finite_set import FiniteSet, fs_difference, fs_intersection,
    finite_set_difference_cardinality_is_sub_intersection,
    finite_set_intersection_cardinality_is_of_subset_right
from data.finite.finite_set_card import fs_card, fs_card_cardinality_is, fs_card_eq_of_cardinality_is

numerals Nat

/// Removing a subset drops the cardinality by exactly the subset's size.
theorem fs_card_difference_of_subset[T](s: FiniteSet[T], b: FiniteSet[T]) {
    b.subset_eq(s) implies fs_card(fs_difference(s, b)) = fs_card(s) - fs_card(b)
} by {
    if b.subset_eq(s) {
        fs_card_cardinality_is(s)
        s.cardinality_is(fs_card(s))
        fs_card_cardinality_is(b)
        b.cardinality_is(fs_card(b))
        finite_set_intersection_cardinality_is_of_subset_right(s, b, fs_card(b))
        fs_intersection(s, b).cardinality_is(fs_card(b))
        finite_set_difference_cardinality_is_sub_intersection(s, b, fs_card(s), fs_card(b))
        fs_difference(s, b).cardinality_is(fs_card(s) - fs_card(b))
        fs_card_eq_of_cardinality_is(fs_difference(s, b), fs_card(s) - fs_card(b))
        fs_card(fs_difference(s, b)) = fs_card(s) - fs_card(b)
    }
}
