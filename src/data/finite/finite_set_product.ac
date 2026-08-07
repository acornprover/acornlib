from finite_set import FiniteSet, fs_empty, fs_from_list, finite_set_ext,
    finite_set_from_unique_list_cardinality_is_length, finite_set_cardinality_has_exact_unique_list
from list import List
from data.list.list_pair_product import list_pair_product, list_pair_product_contains_of_contains,
    list_pair_product_contains_imp_left, list_pair_product_contains_imp_right,
    list_pair_product_unique, list_pair_product_length
from nat import Nat
from pair import Pair, pair_eta
from data.basic.pair_set import pair_set, pair_set_contains_eq, pair_set_contains_imp_left,
    pair_set_contains_imp_right
from data.basic.set import Set, finite_constraint, contains_set_intro, set_ext, empty_set_contains_eq,
    list_set, list_set_contains_eq, subset_contains, subset_contains_eq

/// The finite Cartesian product of two finite sets.
let finite_set_product[T, U](a: FiniteSet[T], b: FiniteSet[U]) -> result: FiniteSet[Pair[T, U]] satisfy {
    FiniteSet.new(pair_set(a.underlying_set, b.underlying_set)) = Option.some(result)
} by {
    finite_constraint(a.underlying_set.contains)
    finite_constraint(b.underlying_set.contains)
    let left_superset: List[T] satisfy {
        forall(x: T) {
            a.underlying_set.contains(x) implies left_superset.contains(x)
        }
    }
    let right_superset: List[U] satisfy {
        forall(y: U) {
            b.underlying_set.contains(y) implies right_superset.contains(y)
        }
    }
    let product_superset = list_pair_product(left_superset, right_superset)
    forall(p: Pair[T, U]) {
        if pair_set(a.underlying_set, b.underlying_set).contains(p) {
            pair_set_contains_imp_left(a.underlying_set, b.underlying_set, p)
            pair_set_contains_imp_right(a.underlying_set, b.underlying_set, p)
            right_superset.contains(p.second)
            list_pair_product_contains_of_contains(left_superset, right_superset, p.first, p.second)
            product_superset.contains(Pair.new(p.first, p.second))
            pair_eta(p)
            product_superset.contains(p)
        }
    }
    contains_set_intro(product_superset, pair_set(a.underlying_set, b.underlying_set))
    finite_constraint(pair_set(a.underlying_set, b.underlying_set).contains)
    pair_set(a.underlying_set, b.underlying_set).is_finite
}

/// The underlying set of the finite Cartesian product is the unbundled Cartesian product.
theorem finite_set_product_underlying_set[T, U](a: FiniteSet[T], b: FiniteSet[U]) {
    finite_set_product(a, b).underlying_set = pair_set(a.underlying_set, b.underlying_set)
}

/// Membership in the finite Cartesian product is componentwise membership.
theorem finite_set_product_contains_eq[T, U](a: FiniteSet[T], b: FiniteSet[U], p: Pair[T, U]) {
    finite_set_product(a, b).contains(p) = (a.contains(p.first) and b.contains(p.second))
} by {
    finite_set_product_underlying_set(a, b)
    pair_set_contains_eq(a.underlying_set, b.underlying_set, p)
}

/// A pair whose components lie in the factors lies in their finite Cartesian product.
theorem finite_set_product_contains_pair[T, U](a: FiniteSet[T], b: FiniteSet[U], x: T, y: U) {
    a.contains(x) and b.contains(y) implies finite_set_product(a, b).contains(Pair.new(x, y))
} by {
    if a.contains(x) and b.contains(y) {
        finite_set_product_contains_eq(a, b, Pair.new(x, y))
        finite_set_product(a, b).contains(Pair.new(x, y))
    }
}

/// Membership in a finite Cartesian product gives membership of the first component.
theorem finite_set_product_contains_left[T, U](a: FiniteSet[T], b: FiniteSet[U], p: Pair[T, U]) {
    finite_set_product(a, b).contains(p) implies a.contains(p.first)
} by {
    if finite_set_product(a, b).contains(p) {
        finite_set_product_contains_eq(a, b, p)
        a.contains(p.first)
    }
}

/// Membership in a finite Cartesian product gives membership of the second component.
theorem finite_set_product_contains_right[T, U](a: FiniteSet[T], b: FiniteSet[U], p: Pair[T, U]) {
    finite_set_product(a, b).contains(p) implies b.contains(p.second)
} by {
    if finite_set_product(a, b).contains(p) {
        finite_set_product_contains_eq(a, b, p)
        b.contains(p.second)
    }
}

/// The product with an empty left factor is empty.
theorem finite_set_product_empty_left[T, U](b: FiniteSet[U]) {
    finite_set_product(fs_empty[T](Set[T].empty_set), b) =
    fs_empty[Pair[T, U]](Set[Pair[T, U]].empty_set)
} by {
    let left_empty = fs_empty[T](Set[T].empty_set)
    let product = finite_set_product(left_empty, b)
    let empty_product = fs_empty[Pair[T, U]](Set[Pair[T, U]].empty_set)
    forall(p: Pair[T, U]) {
        finite_set_product_contains_eq(left_empty, b, p)
        left_empty.underlying_set = Set[T].empty_set
        empty_set_contains_eq[T](p.first)
        product.contains(p) = false
        empty_set_contains_eq[Pair[T, U]](p)
        empty_product.underlying_set = Set[Pair[T, U]].empty_set
        empty_product.underlying_set.contains(p) = false
        product.underlying_set.contains(p) = product.contains(p)
        product.underlying_set.contains(p) = empty_product.underlying_set.contains(p)
    }
    set_ext(product.underlying_set, empty_product.underlying_set)
    finite_set_ext(product, empty_product)
}

/// The product with an empty right factor is empty.
theorem finite_set_product_empty_right[T, U](a: FiniteSet[T]) {
    finite_set_product(a, fs_empty[U](Set[U].empty_set)) =
    fs_empty[Pair[T, U]](Set[Pair[T, U]].empty_set)
} by {
    let right_empty = fs_empty[U](Set[U].empty_set)
    let product = finite_set_product(a, right_empty)
    let empty_product = fs_empty[Pair[T, U]](Set[Pair[T, U]].empty_set)
    forall(p: Pair[T, U]) {
        finite_set_product_contains_eq(a, right_empty, p)
        right_empty.underlying_set = Set[U].empty_set
        empty_set_contains_eq[U](p.second)
        right_empty.contains(p.second) = false
        product.contains(p) = false
        empty_set_contains_eq[Pair[T, U]](p)
        empty_product.underlying_set = Set[Pair[T, U]].empty_set
        empty_product.underlying_set.contains(p) = false
        product.underlying_set.contains(p) = product.contains(p)
        product.underlying_set.contains(p) = empty_product.underlying_set.contains(p)
    }
    set_ext(product.underlying_set, empty_product.underlying_set)
    finite_set_ext(product, empty_product)
}

/// Cartesian product is monotone in both finite-set arguments.
theorem finite_set_product_subset[T, U](a: FiniteSet[T], a_target: FiniteSet[T],
    b: FiniteSet[U], b_target: FiniteSet[U]) {
    a.subset_eq(a_target) and b.subset_eq(b_target) implies
    finite_set_product(a, b).subset_eq(finite_set_product(a_target, b_target))
} by {
    if a.subset_eq(a_target) and b.subset_eq(b_target) {
        let source = finite_set_product(a, b)
        let target = finite_set_product(a_target, b_target)
        forall(p: Pair[T, U]) {
            if source.underlying_set.contains(p) {
                source.contains(p)
                finite_set_product_contains_left(a, b, p)
                finite_set_product_contains_right(a, b, p)
                a.contains(p.first)
                b.contains(p.second)
                a.subset_eq(a_target) = a.underlying_set.subset(a_target.underlying_set)
                b.subset_eq(b_target) = b.underlying_set.subset(b_target.underlying_set)
                subset_contains(a.underlying_set, a_target.underlying_set, p.first)
                subset_contains(b.underlying_set, b_target.underlying_set, p.second)
                a_target.contains(p.first)
                b_target.contains(p.second)
                finite_set_product_contains_pair(a_target, b_target, p.first, p.second)
                target.contains(Pair.new(p.first, p.second))
                pair_eta(p)
                target.contains(p)
                target.underlying_set.contains(p)
            }
        }
        forall(q: Pair[T, U]) {
            source.underlying_set.contains(q) implies target.underlying_set.contains(q)
        }
        subset_contains_eq(source.underlying_set, target.underlying_set)
        source.underlying_set.subset(target.underlying_set) = forall(q: Pair[T, U]) {
            source.underlying_set.contains(q) implies target.underlying_set.contains(q)
        }
        source.underlying_set.subset(target.underlying_set)
        source.subset_eq(target) = source.underlying_set.subset(target.underlying_set)
        source.subset_eq(target)
        finite_set_product(a, b).subset_eq(finite_set_product(a_target, b_target))
    }
}

/// Membership in a finite set built from a list is list membership.
theorem finite_set_product_fs_from_list_contains_eq[T](items: List[T], x: T) {
    fs_from_list(items).contains(x) = items.contains(x)
} by {
    fs_from_list(items).contains(x) = fs_from_list(items).underlying_set.contains(x)
    fs_from_list(items).underlying_set = list_set(items)
    fs_from_list(items).underlying_set.contains(x) = list_set(items).contains(x)
    list_set_contains_eq(items, x)
    list_set(items).contains(x) = items.contains(x)
}

/// List-product membership maps into the finite product of the listed factors.
theorem list_pair_product_contains_imp_finite_set_product_from_list_contains[T, U](left: List[T],
    right: List[U], p: Pair[T, U]) {
    list_pair_product(left, right).contains(p) implies
    finite_set_product(fs_from_list(left), fs_from_list(right)).contains(p)
} by {
    if list_pair_product(left, right).contains(p) {
        list_pair_product_contains_imp_left(left, right, p)
        list_pair_product_contains_imp_right(left, right, p)
        left.contains(p.first)
        right.contains(p.second)
        finite_set_product_fs_from_list_contains_eq(left, p.first)
        fs_from_list(left).contains(p.first)
        finite_set_product_fs_from_list_contains_eq(right, p.second)
        fs_from_list(right).contains(p.second)
        finite_set_product_contains_eq(fs_from_list(left), fs_from_list(right), p)
        finite_set_product(fs_from_list(left), fs_from_list(right)).contains(p)
    }
}

/// The finite product of listed factors maps into list-product membership.
theorem finite_set_product_from_list_contains_imp_list_pair_product_contains[T, U](left: List[T],
    right: List[U], p: Pair[T, U]) {
    finite_set_product(fs_from_list(left), fs_from_list(right)).contains(p) implies
    list_pair_product(left, right).contains(p)
} by {
    if finite_set_product(fs_from_list(left), fs_from_list(right)).contains(p) {
        finite_set_product_contains_eq(fs_from_list(left), fs_from_list(right), p)
        fs_from_list(left).contains(p.first)
        fs_from_list(right).contains(p.second)
        finite_set_product_fs_from_list_contains_eq(left, p.first)
        left.contains(p.first)
        finite_set_product_fs_from_list_contains_eq(right, p.second)
        right.contains(p.second)
        list_pair_product_contains_of_contains(left, right, p.first, p.second)
        list_pair_product(left, right).contains(Pair.new(p.first, p.second))
        pair_eta(p)
        list_pair_product(left, right).contains(p)
    }
}

/// The finite product of list-backed finite sets is represented by the list product.
theorem finite_set_product_from_list_eq[T, U](left: List[T], right: List[U]) {
    finite_set_product(fs_from_list(left), fs_from_list(right)) =
    fs_from_list(list_pair_product(left, right))
} by {
    let product = finite_set_product(fs_from_list(left), fs_from_list(right))
    let product_items = list_pair_product(left, right)
    let listed = fs_from_list(product_items)
    forall(p: Pair[T, U]) {
        if listed.underlying_set.contains(p) {
            listed.contains(p)
            finite_set_product_fs_from_list_contains_eq(product_items, p)
            product_items.contains(p)
            list_pair_product_contains_imp_finite_set_product_from_list_contains(left, right, p)
            product.contains(p)
            product.underlying_set.contains(p)
        }
        if product.underlying_set.contains(p) {
            product.contains(p)
            finite_set_product_from_list_contains_imp_list_pair_product_contains(left, right, p)
            product_items.contains(p)
            finite_set_product_fs_from_list_contains_eq(product_items, p)
            listed.contains(p)
            listed.underlying_set.contains(p)
        }
        product.underlying_set.contains(p) = listed.underlying_set.contains(p)
    }
    set_ext(product.underlying_set, listed.underlying_set)
    product.underlying_set = listed.underlying_set
    finite_set_ext(product, listed)
    product = listed
}

/// Exact cardinality of a finite Cartesian product.
theorem finite_set_product_cardinality_is[T, U](a: FiniteSet[T], b: FiniteSet[U],
    n_left: Nat, n_right: Nat) {
    a.cardinality_is(n_left) and b.cardinality_is(n_right) implies
    finite_set_product(a, b).cardinality_is(n_right * n_left)
} by {
    if a.cardinality_is(n_left) and b.cardinality_is(n_right) {
        finite_set_cardinality_has_exact_unique_list(a, n_left)
        let left_items: List[T] satisfy {
            fs_from_list(left_items) = a and left_items.is_unique and left_items.length = n_left
        }
        finite_set_cardinality_has_exact_unique_list(b, n_right)
        let right_items: List[U] satisfy {
            fs_from_list(right_items) = b and right_items.is_unique and right_items.length = n_right
        }
        let product_items = list_pair_product(left_items, right_items)
        list_pair_product_unique(left_items, right_items)
        product_items.is_unique
        finite_set_from_unique_list_cardinality_is_length(product_items)
        fs_from_list(product_items).cardinality_is(product_items.length)
        list_pair_product_length(left_items, right_items)
        product_items.length = right_items.length * left_items.length
        product_items.length = n_right * n_left
        fs_from_list(product_items).cardinality_is(n_right * n_left)
        finite_set_product_from_list_eq(left_items, right_items)
        finite_set_product(fs_from_list(left_items), fs_from_list(right_items)) = fs_from_list(product_items)
        fs_from_list(left_items) = a
        fs_from_list(right_items) = b
        finite_set_product(a, b) = fs_from_list(product_items)
        finite_set_product(a, b).cardinality_is(n_right * n_left)
    }
}
