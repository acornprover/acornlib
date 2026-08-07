from nat import Nat, sub_self
from finite_set import FiniteSet, fs_from_list, fs_difference,
    finite_set_cardinality_is_smallest_cardinality, finite_set_subset_antisymm,
    finite_set_cardinality_has_exact_unique_list, finite_set_difference_contains_eq,
    finite_set_intersection_cardinality_is_of_subset_right,
    finite_set_difference_cardinality_is_sub_intersection
from data.basic.set import subset_cardinality_at_most, list_set, list_set_contains_eq, subset_contains_eq
from list import List

/// A finite subset has cardinality at most the exact cardinality of a finite superset.
theorem finite_set_subset_cardinality_at_most[T](s: FiniteSet[T], t: FiniteSet[T], n: Nat) {
    s.subset_eq(t) and t.cardinality_is(n) implies s.cardinality_at_most(n)
} by {
    if s.subset_eq(t) and t.cardinality_is(n) {
        finite_set_cardinality_is_smallest_cardinality(t, n)
        t.underlying_set.cardinality_at_most(n)
        subset_cardinality_at_most(s.underlying_set, t.underlying_set, n)
        s.underlying_set.cardinality_at_most(n)
        s.cardinality_at_most(n)
    }
}

/// A finite subset inherits any cardinality upper bound from a finite superset.
theorem finite_set_subset_cardinality_at_most_of_at_most[T](s: FiniteSet[T], t: FiniteSet[T], n: Nat) {
    s.subset_eq(t) and t.cardinality_at_most(n) implies s.cardinality_at_most(n)
} by {
    if s.subset_eq(t) and t.cardinality_at_most(n) {
        t.underlying_set.cardinality_at_most(n)
        subset_cardinality_at_most(s.underlying_set, t.underlying_set, n)
        s.underlying_set.cardinality_at_most(n)
        s.cardinality_at_most(n)
    }
}

/// A finite subset with the same exact cardinality as its finite superset is equal to it.
theorem finite_set_subset_eq_of_same_cardinality[T](s: FiniteSet[T], t: FiniteSet[T], n: Nat) {
    s.subset_eq(t) and s.cardinality_is(n) and t.cardinality_is(n) implies s = t
} by {
    if s.subset_eq(t) and s.cardinality_is(n) and t.cardinality_is(n) {
        finite_set_intersection_cardinality_is_of_subset_right(t, s, n)
        finite_set_difference_cardinality_is_sub_intersection(t, s, n, n)
        fs_difference(t, s).cardinality_is(n - n)
        sub_self(n)
        fs_difference(t, s).cardinality_is(Nat.0)
        finite_set_cardinality_has_exact_unique_list(fs_difference(t, s), Nat.0)
        let diff_items: List[T] satisfy {
            fs_from_list(diff_items) = fs_difference(t, s) and diff_items.is_unique and diff_items.length = Nat.0
        }
        match diff_items {
            List.nil {
            }
            List.cons(head, tail) {
                tail.length.suc != Nat.0
                false
            }
        }
        diff_items = List.nil[T]
        forall(x: T) {
            if t.underlying_set.contains(x) {
                t.contains(x)
                if not s.contains(x) {
                    finite_set_difference_contains_eq(t, s, x)
                    fs_difference(t, s).contains(x)
                    fs_from_list(diff_items).contains(x)
                    fs_from_list(diff_items).underlying_set = list_set(diff_items)
                    list_set_contains_eq(diff_items, x)
                    not List.nil[T].contains(x)
                    false
                }
                s.underlying_set.contains(x)
            }
        }
        subset_contains_eq(t.underlying_set, s.underlying_set)
        t.subset_eq(s)
        finite_set_subset_antisymm(s, t)
        s = t
    }
}
