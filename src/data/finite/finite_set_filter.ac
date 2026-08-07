from finite_set import FiniteSet, finite_set_ext, finite_set_empty_contains_eq
from data.basic.set import Set, set_ext, intersection_contains_eq, sets_subset_intersection,
    subset_trans, set_supset_contains_intersection, subset_is_finite_of_finite

/// The finite subset of `s` cut out by the predicate `p`.
let finite_set_filter[T](s: FiniteSet[T], p: T -> Bool) -> result: FiniteSet[T] satisfy {
    FiniteSet.new(s.underlying_set.intersection(Set[T].new(p))) = Option.some(result)
} by {
    let target = s.underlying_set.intersection(Set[T].new(p))
    forall(x: T) {
        if target.contains(x) {
            intersection_contains_eq(s.underlying_set, Set[T].new(p), x)
            s.underlying_set.contains(x)
        }
    }
    target.subset(s.underlying_set)
    subset_is_finite_of_finite(target, s.underlying_set)
    target.is_finite
}

/// The filter remembers its defining underlying set.
theorem finite_set_filter_underlying_set[T](s: FiniteSet[T], p: T -> Bool) {
    finite_set_filter(s, p).underlying_set = s.underlying_set.intersection(Set[T].new(p))
} by {
}

/// Membership in a finite-set filter is membership in the source and satisfaction of the predicate.
theorem finite_set_filter_contains_eq[T](s: FiniteSet[T], p: T -> Bool, x: T) {
    finite_set_filter(s, p).contains(x) = (s.contains(x) and p(x))
} by {
    finite_set_filter_underlying_set(s, p)
    intersection_contains_eq(s.underlying_set, Set[T].new(p), x)
}

/// A filtered finite set is a subset of the source finite set.
theorem finite_set_filter_subset[T](s: FiniteSet[T], p: T -> Bool) {
    finite_set_filter(s, p).subset_eq(s)
} by {
    forall(x: T) {
        if finite_set_filter(s, p).contains(x) {
            finite_set_filter_contains_eq(s, p, x)
            s.contains(x)
        }
    }
}

/// Finite-set filtering is monotone in the predicate.
theorem finite_set_filter_mono_pred[T](s: FiniteSet[T], p: T -> Bool, q: T -> Bool) {
    (forall(x: T) { p(x) implies q(x) }) implies
    finite_set_filter(s, p).subset_eq(finite_set_filter(s, q))
} by {
    if forall(x: T) { p(x) implies q(x) } {
        let fp = finite_set_filter(s, p)
        let fq = finite_set_filter(s, q)
        let pset = Set[T].new(p)
        let qset = Set[T].new(q)

        finite_set_filter_underlying_set(s, p)
        finite_set_filter_underlying_set(s, q)

        forall(x: T) {
            if pset.contains(x) {
                p(x)
                qset.contains(x)
            }
        }

        sets_subset_intersection(s.underlying_set, pset)
        subset_trans(s.underlying_set.intersection(pset), pset, qset)
        s.underlying_set.intersection(pset).subset(qset)

        qset.superset(s.underlying_set.intersection(pset))
        set_supset_contains_intersection(s.underlying_set, qset, s.underlying_set.intersection(pset))
        s.underlying_set.intersection(qset).superset(s.underlying_set.intersection(pset))
        fp.subset_eq(fq)
        finite_set_filter(s, p).subset_eq(finite_set_filter(s, q))
    }
}

/// Filtering by the always-true predicate preserves the finite set.
theorem finite_set_filter_true[T](s: FiniteSet[T]) {
    finite_set_filter(s, function(y: T) { true }) = s
} by {
    let filtered = finite_set_filter(s, function(y: T) { true })
    forall(x: T) {
        finite_set_filter_contains_eq(s, function(y: T) { true }, x)
        filtered.contains(x) = s.contains(x)
        filtered.underlying_set.contains(x) = s.underlying_set.contains(x)
    }
    set_ext(filtered.underlying_set, s.underlying_set)
    filtered.underlying_set = s.underlying_set
    finite_set_ext(filtered, s)
}

/// Filtering by the always-false predicate gives the empty finite set.
theorem finite_set_filter_false[T](s: FiniteSet[T]) {
    finite_set_filter(s, function(y: T) { false }) = FiniteSet.empty[T]
} by {
    let filtered = finite_set_filter(s, function(y: T) { false })
    forall(x: T) {
        finite_set_filter_contains_eq(s, function(y: T) { false }, x)
        finite_set_empty_contains_eq[T](x)
        FiniteSet.empty[T].underlying_set.contains(x) = false
        filtered.underlying_set.contains(x) = FiniteSet.empty[T].underlying_set.contains(x)
    }
    set_ext(filtered.underlying_set, FiniteSet.empty[T].underlying_set)
    filtered.underlying_set = FiniteSet.empty[T].underlying_set
    finite_set_ext(filtered, FiniteSet.empty[T])
}

/// Filtering twice by the same predicate is the same as filtering once.
theorem finite_set_filter_idempotent[T](s: FiniteSet[T], p: T -> Bool) {
    finite_set_filter(finite_set_filter(s, p), p) = finite_set_filter(s, p)
} by {
    let filtered = finite_set_filter(s, p)
    let twice = finite_set_filter(filtered, p)
    forall(x: T) {
        finite_set_filter_contains_eq(filtered, p, x)
        finite_set_filter_contains_eq(s, p, x)
        if twice.contains(x) {
            filtered.contains(x)
        }
        if filtered.contains(x) {
            twice.contains(x)
        }
        twice.contains(x) = filtered.contains(x)
        twice.underlying_set.contains(x) = filtered.underlying_set.contains(x)
    }
    set_ext(twice.underlying_set, filtered.underlying_set)
    twice.underlying_set = filtered.underlying_set
    finite_set_ext(twice, filtered)
}
