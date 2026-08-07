from nat import Nat, lte_trans
from list import List, unique_length, filter_only_removes_elems
from data.basic.set import Set, cardinality_is_well_defined
from finite_set import FiniteSet, finite_set_cardinality_is_well_defined
from data.finite.finite_set_card import fs_card, fs_card_eq_of_cardinality_is, fs_card_cardinality_is
from data.finite.finite_set_subset_cardinality import finite_set_subset_cardinality_at_most

numerals Nat

/// A bound on the cardinality of a set bounds its exact cardinality.
///
/// `cardinality_at_most` asserts a containing list of bounded length, while
/// `cardinality_is` counts the distinct members of such a list. Filtering and
/// deduplicating only shorten a list, so the exact count is below any such bound.
theorem cardinality_is_le_of_at_most[K](s: Set[K], m: Nat, n: Nat) {
    s.cardinality_at_most(n) and s.cardinality_is(m) implies m <= n
} by {
    if s.cardinality_at_most(n) and s.cardinality_is(m) {
        s.cardinality_at_most(n) = exists(superset: List[K]) {
            superset.contains_set(s) and superset.length <= n
        }
        exists(superset: List[K]) {
            superset.contains_set(s) and superset.length <= n
        }
        let (sup: List[K]) satisfy {
            sup.contains_set(s) and sup.length <= n
        }
        s.cardinality_is(sup.filter(s.contains).unique.length) = exists(containing_list: List[K]) {
            containing_list.contains_set(s) and containing_list.filter(s.contains).unique.length = sup.filter(s.contains).unique.length
        }
        exists(containing_list: List[K]) {
            containing_list.contains_set(s) and containing_list.filter(s.contains).unique.length = sup.filter(s.contains).unique.length
        }
        s.cardinality_is(sup.filter(s.contains).unique.length)
        cardinality_is_well_defined(s, m, sup.filter(s.contains).unique.length)
        m = sup.filter(s.contains).unique.length
        unique_length(sup.filter(s.contains))
        sup.filter(s.contains).unique.length <= sup.filter(s.contains).length
        filter_only_removes_elems(sup, s.contains)
        sup.filter(s.contains).length <= sup.length
        lte_trans(sup.filter(s.contains).unique.length, sup.filter(s.contains).length, sup.length)
        sup.filter(s.contains).unique.length <= sup.length
        lte_trans(sup.filter(s.contains).unique.length, sup.length, n)
        sup.filter(s.contains).unique.length <= n
        m <= n
    }
}

/// The same bound, stated for finite sets.
theorem finite_set_cardinality_is_le_of_at_most[T](s: FiniteSet[T], m: Nat, n: Nat) {
    s.cardinality_at_most(n) and s.cardinality_is(m) implies m <= n
} by {
    if s.cardinality_at_most(n) and s.cardinality_is(m) {
        s.underlying_set.cardinality_at_most(n)
        s.underlying_set.cardinality_is(m)
        cardinality_is_le_of_at_most(s.underlying_set, m, n)
        m <= n
    }
}

/// The count of a finite set is below any bound on it.
theorem fs_card_le_of_at_most[T](s: FiniteSet[T], n: Nat) {
    s.cardinality_at_most(n) implies fs_card(s) <= n
} by {
    if s.cardinality_at_most(n) {
        fs_card_cardinality_is(s)
        s.cardinality_is(fs_card(s))
        finite_set_cardinality_is_le_of_at_most(s, fs_card(s), n)
        fs_card(s) <= n
    }
}

/// Counting is monotone under inclusion.
///
/// This is the lemma the union, intersection, and difference bounds all rest on.
theorem fs_card_mono[T](s: FiniteSet[T], t: FiniteSet[T]) {
    s.subset_eq(t) implies fs_card(s) <= fs_card(t)
} by {
    if s.subset_eq(t) {
        fs_card_cardinality_is(t)
        t.cardinality_is(fs_card(t))
        finite_set_subset_cardinality_at_most(s, t, fs_card(t))
        s.cardinality_at_most(fs_card(t))
        fs_card_le_of_at_most(s, fs_card(t))
        fs_card(s) <= fs_card(t)
    }
}
