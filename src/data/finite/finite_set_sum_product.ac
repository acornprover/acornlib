from algebra.add_comm_monoid import AddCommMonoid
from finite_set import FiniteSet, fs_from_list, finite_set_has_unique_list,
    finite_set_sum, finite_set_sum_eq_list_sum
from data.finite.finite_set_product import finite_set_product, finite_set_product_from_list_eq
from data.basic.functions import compose, function_extensionality
from list import List, map, sum, map_add, sum_add, map_map
from data.list.list_pair_product import pair_with_left, pair_with_left_fn, list_pair_with_left,
    list_pair_product, list_pair_product_unique
from pair import Pair

/// The row of a function on a Cartesian product obtained by fixing the left coordinate.
define finite_set_product_row_fn[T, U, A](f: Pair[T, U] -> A, x: T) -> (U -> A) {
    function(y: U) { f(Pair.new(x, y)) }
}

/// The row-sum function over a list of right coordinates.
define list_pair_product_row_sum_fn[T, U, A: AddCommMonoid](right: List[U],
    f: Pair[T, U] -> A) -> (T -> A) {
    function(x: T) { sum[A](map(right, finite_set_product_row_fn(f, x))) }
}

/// The mapped sum over one fixed-left product row is the corresponding row sum.
theorem list_pair_with_left_sum_eq_row_sum[T, U, A: AddCommMonoid](right: List[U],
    f: Pair[T, U] -> A, x: T) {
    sum[A](map(list_pair_with_left(x, right), f)) =
        sum[A](map(right, finite_set_product_row_fn(f, x)))
} by {
    map_map[U, Pair[T, U], A](right, pair_with_left_fn[T, U](x), f)
    forall(y: U) {
        pair_with_left(x, y) = Pair.new(x, y)
        finite_set_product_row_fn(f, x, y) = f(Pair.new(x, y))
        compose[U, Pair[T, U], A](f, pair_with_left_fn[T, U](x), y) =
            finite_set_product_row_fn(f, x, y)
    }
    function_extensionality[U, A](compose(f, pair_with_left_fn[T, U](x)),
        finite_set_product_row_fn(f, x))
    compose(f, pair_with_left_fn[T, U](x)) = finite_set_product_row_fn(f, x)
    map(right, compose(f, pair_with_left_fn[T, U](x))) =
        map(right, finite_set_product_row_fn(f, x))
    map(list_pair_with_left(x, right), f) = map(right, finite_set_product_row_fn(f, x))
    sum[A](map(list_pair_with_left(x, right), f)) =
        sum[A](map(right, finite_set_product_row_fn(f, x)))
}

/// The flat sum over a list Cartesian product is the iterated sum over its rows.
theorem list_pair_product_fubini_rows[T, U, A: AddCommMonoid](left: List[T], right: List[U],
    f: Pair[T, U] -> A) {
    sum[A](map(list_pair_product(left, right), f)) =
        sum[A](map(left, list_pair_product_row_sum_fn(right, f)))
} by {
    define p(items: List[T]) -> Bool {
        sum[A](map(items, list_pair_product_row_sum_fn(right, f))) =
            sum[A](map(list_pair_product(items, right), f))
    }

    list_pair_product(List.nil[T], right) = List.nil[Pair[T, U]]
    map(List.nil[Pair[T, U]], f) = List.nil[A]
    map(List.nil[T], list_pair_product_row_sum_fn(right, f)) = List.nil[A]
    sum[A](List.nil[A]) = A.0
    p(List.nil[T])

    forall(head: T, tail: List[T]) {
        if p(tail) {
            let row = list_pair_with_left(head, right)
            let rest = list_pair_product(tail, right)
            let row_sum = list_pair_product_row_sum_fn(right, f)
            list_pair_product(List.cons(head, tail), right) = row + rest
            map_add[Pair[T, U], A](row, rest, f)
            map(row + rest, f) = map(row, f) + map(rest, f)
            sum_add[A](map(row, f), map(rest, f))
            sum[A](map(row, f) + map(rest, f)) =
                sum[A](map(row, f)) + sum[A](map(rest, f))
            sum[A](map(list_pair_product(List.cons(head, tail), right), f)) =
                sum[A](map(row, f)) + sum[A](map(rest, f))
            list_pair_with_left_sum_eq_row_sum(right, f, head)
            sum[A](map(row, f)) = sum[A](map(right, finite_set_product_row_fn(f, head)))
            row_sum(head) = sum[A](map(right, finite_set_product_row_fn(f, head)))
            sum[A](map(row, f)) = row_sum(head)
            p(tail) =
                (sum[A](map(tail, row_sum)) = sum[A](map(list_pair_product(tail, right), f)))
            sum[A](map(tail, row_sum)) = sum[A](map(list_pair_product(tail, right), f))
            sum[A](map(list_pair_product(tail, right), f)) = sum[A](map(tail, row_sum))
            rest = list_pair_product(tail, right)
            sum[A](map(rest, f)) = sum[A](map(tail, row_sum))
            sum[A](map(list_pair_product(List.cons(head, tail), right), f)) =
                row_sum(head) + sum[A](map(tail, row_sum))
            map(List.cons(head, tail), row_sum) = List.cons(row_sum(head), map(tail, row_sum))
            sum[A](List.cons(row_sum(head), map(tail, row_sum))) =
                row_sum(head) + sum[A](map(tail, row_sum))
            sum[A](map(List.cons(head, tail), row_sum)) =
                row_sum(head) + sum[A](map(tail, row_sum))
            sum[A](map(list_pair_product(List.cons(head, tail), right), f)) =
                sum[A](map(List.cons(head, tail), row_sum))
            p(List.cons(head, tail))
        }
    }
    List.induction(function(items: List[T]) { p(items) })
    function[T0](x0: List[T0] -> Bool, x1: List[T0]) {
        not x0(List.nil[T0]) or exists(k0: T0, k1: List[T0]) {
            x0(k1) and not x0(List.cons[T0](k0, k1))
        } or x0(x1)
    }[T](p, left)
    p(left)
}

/// Finite-set sums are transported across equality of finite sets.
theorem finite_set_sum_eq_of_finite_set_eq[T, A: AddCommMonoid](s: FiniteSet[T], t: FiniteSet[T],
    f: T -> A) {
    s = t implies finite_set_sum(s, f) = finite_set_sum(t, f)
}

/// Finite-set sums are transported across equality of summand functions.
theorem finite_set_sum_eq_of_function_eq[T, A: AddCommMonoid](s: FiniteSet[T],
    f: T -> A, g: T -> A) {
    f = g implies finite_set_sum(s, f) = finite_set_sum(s, g)
}

/// For unique list representatives, the finite-set sum over their Cartesian
/// product is the sum over the corresponding list Cartesian product.
theorem finite_set_sum_product_from_lists_eq_list_pair_product_sum[T, U, A: AddCommMonoid](
    left: List[T], right: List[U], f: Pair[T, U] -> A
) {
    left.is_unique and right.is_unique implies
    finite_set_sum(finite_set_product(fs_from_list(left), fs_from_list(right)), f) =
    sum[A](map(list_pair_product(left, right), f))
} by {
    if left.is_unique and right.is_unique {
        let product_items = list_pair_product(left, right)
        let product = finite_set_product(fs_from_list(left), fs_from_list(right))
        let listed = fs_from_list(product_items)
        let rhs = sum[A](map(product_items, f))
        list_pair_product_unique(left, right)
        product_items.is_unique
        finite_set_sum_eq_list_sum[Pair[T, U], A](product_items, f)
        finite_set_sum(listed, f) = rhs
        finite_set_product_from_list_eq(left, right)
        product = listed
        finite_set_sum_eq_of_finite_set_eq(product, listed, f)
        finite_set_sum(product, f) = finite_set_sum(listed, f)
        finite_set_sum(product, f) = rhs
        product = finite_set_product(fs_from_list(left), fs_from_list(right))
        product_items = list_pair_product(left, right)
        rhs = sum[A](map(list_pair_product(left, right), f))
        finite_set_sum(finite_set_product(fs_from_list(left), fs_from_list(right)), f) = rhs
        finite_set_sum(finite_set_product(fs_from_list(left), fs_from_list(right)), f) =
            sum[A](map(list_pair_product(left, right), f))
    }
}

/// Finite-set Fubini theorem for additive sums over Cartesian products, expanded by rows.
theorem finite_set_sum_product_fubini_rows[T, U, A: AddCommMonoid](
    a: FiniteSet[T], b: FiniteSet[U], f: Pair[T, U] -> A
) {
    finite_set_sum(finite_set_product(a, b), f) =
        finite_set_sum(a, function(x: T) { finite_set_sum(b, function(y: U) { f(Pair.new(x, y)) }) })
} by {
    finite_set_has_unique_list(a)
    let left: List[T] satisfy {
        fs_from_list(left) = a and left.is_unique
    }
    finite_set_has_unique_list(b)
    let right: List[U] satisfy {
        fs_from_list(right) = b and right.is_unique
    }

    let product = finite_set_product(a, b)
    let flat_sum = sum[A](map(list_pair_product(left, right), f))
    let row_sum = list_pair_product_row_sum_fn(right, f)
    let nested_set_sum: T -> A = function(x: T) {
        finite_set_sum(b, function(y: U) { f(Pair.new(x, y)) })
    }

    finite_set_sum_product_from_lists_eq_list_pair_product_sum(left, right, f)
    finite_set_sum(finite_set_product(fs_from_list(left), fs_from_list(right)), f) = flat_sum
    fs_from_list(left) = a
    fs_from_list(right) = b
    product = finite_set_product(fs_from_list(left), fs_from_list(right))
    finite_set_sum(product, f) = flat_sum

    list_pair_product_fubini_rows(left, right, f)
    flat_sum = sum[A](map(left, row_sum))

    forall(x: T) {
        let inline_row: U -> A = function(y: U) { f(Pair.new(x, y)) }
        forall(y: U) {
            inline_row(y) = f(Pair.new(x, y))
            finite_set_product_row_fn(f, x, y) = f(Pair.new(x, y))
            inline_row(y) = finite_set_product_row_fn(f, x, y)
        }
        function_extensionality[U, A](inline_row, finite_set_product_row_fn(f, x))
        inline_row = finite_set_product_row_fn(f, x)
        finite_set_sum(b, inline_row) = finite_set_sum(b, finite_set_product_row_fn(f, x))
        finite_set_sum_eq_list_sum[U, A](right, finite_set_product_row_fn(f, x))
        finite_set_sum(fs_from_list(right), finite_set_product_row_fn(f, x)) =
            sum[A](map(right, finite_set_product_row_fn(f, x)))
        finite_set_sum(b, finite_set_product_row_fn(f, x)) =
            sum[A](map(right, finite_set_product_row_fn(f, x)))
        row_sum(x) = sum[A](map(right, finite_set_product_row_fn(f, x)))
        finite_set_sum(b, function(y: U) { f(Pair.new(x, y)) }) =
            finite_set_sum(b, inline_row)
        finite_set_sum(b, function(y: U) { f(Pair.new(x, y)) }) =
            finite_set_sum(b, finite_set_product_row_fn(f, x))
        nested_set_sum(x) = finite_set_sum(b, function(y: U) { f(Pair.new(x, y)) })
        nested_set_sum(x) = row_sum(x)
    }
    function_extensionality[T, A](nested_set_sum, row_sum)
    nested_set_sum = row_sum

    finite_set_sum_eq_list_sum[T, A](left, nested_set_sum)
    finite_set_sum(fs_from_list(left), nested_set_sum) = sum[A](map(left, nested_set_sum))
    finite_set_sum(a, nested_set_sum) = sum[A](map(left, nested_set_sum))
    sum[A](map(left, nested_set_sum)) = sum[A](map(left, row_sum))
    finite_set_sum(a, nested_set_sum) = sum[A](map(left, row_sum))
    finite_set_sum(product, f) = finite_set_sum(a, nested_set_sum)
    let inline_nested: T -> A = function(x: T) {
        finite_set_sum(b, function(y: U) { f(Pair.new(x, y)) })
    }
    nested_set_sum = inline_nested
    finite_set_sum(a, nested_set_sum) = finite_set_sum(a, inline_nested)
    finite_set_sum(product, f) = finite_set_sum(a, inline_nested)
    forall(x: T) {
        inline_nested(x) = finite_set_sum(b, function(y: U) { f(Pair.new(x, y)) })
    }
    function_extensionality[T, A](inline_nested,
        function(x: T) { finite_set_sum(b, function(y: U) { f(Pair.new(x, y)) }) })
    inline_nested = function(x: T) { finite_set_sum(b, function(y: U) { f(Pair.new(x, y)) }) }
    finite_set_sum_eq_of_function_eq(a, inline_nested,
        function(x: T) { finite_set_sum(b, function(y: U) { f(Pair.new(x, y)) }) })
    finite_set_sum(a, inline_nested) =
        finite_set_sum(a, function(x: T) { finite_set_sum(b, function(y: U) { f(Pair.new(x, y)) }) })
    product = finite_set_product(a, b)
    finite_set_sum(finite_set_product(a, b), f) = finite_set_sum(product, f)
    finite_set_sum(finite_set_product(a, b), f) =
        finite_set_sum(a, function(x: T) { finite_set_sum(b, function(y: U) { f(Pair.new(x, y)) }) })
}
