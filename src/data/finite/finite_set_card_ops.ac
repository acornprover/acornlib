from nat import Nat, add_one_right, alt_suc_ne_zero, pos_of_ne_zero
from finite_set import FiniteSet, fs_insert, fs_remove,
    finite_set_insert_cardinality_is_suc_of_not_contains,
    finite_set_remove_insert_cardinality_is_suc_of_contains,
    finite_set_remove_cardinality_is_of_not_contains
from data.finite.finite_set_card import fs_card, fs_card_eq_of_cardinality_is, fs_card_cardinality_is

numerals Nat

/// Inserting a new element increases the count by one.
theorem fs_card_insert_of_not_contains[T](s: FiniteSet[T], item: T) {
    not s.contains(item) implies fs_card(fs_insert(s, item)) = fs_card(s) + Nat.1
} by {
    if not s.contains(item) {
        fs_card_cardinality_is(s)
        finite_set_insert_cardinality_is_suc_of_not_contains(s, item, fs_card(s))
        fs_insert(s, item).cardinality_is(fs_card(s) + Nat.1)
        fs_card_eq_of_cardinality_is(fs_insert(s, item), fs_card(s) + Nat.1)
    }
}

/// Removing an absent element leaves the count unchanged.
theorem fs_card_remove_of_not_contains[T](s: FiniteSet[T], item: T) {
    not s.contains(item) implies fs_card(fs_remove(s, item)) = fs_card(s)
} by {
    if not s.contains(item) {
        fs_card_cardinality_is(s)
        finite_set_remove_cardinality_is_of_not_contains(s, item, fs_card(s))
        fs_remove(s, item).cardinality_is(fs_card(s))
        fs_card_eq_of_cardinality_is(fs_remove(s, item), fs_card(s))
    }
}

/// A set containing an element has one more element than the set without it.
theorem fs_remove_cardinality_is_suc[T](s: FiniteSet[T], item: T) {
    s.contains(item) implies s.cardinality_is(fs_card(fs_remove(s, item)) + Nat.1)
} by {
    if s.contains(item) {
        fs_card_cardinality_is(fs_remove(s, item))
        fs_remove(s, item).cardinality_is(fs_card(fs_remove(s, item)))
        finite_set_remove_insert_cardinality_is_suc_of_contains(
            s, item, fs_card(fs_remove(s, item))
        )
        s.cardinality_is(fs_card(fs_remove(s, item)) + Nat.1)
    }
}

/// Removing a present element decreases the count by one.
theorem fs_card_remove_of_contains[T](s: FiniteSet[T], item: T) {
    s.contains(item) implies fs_card(s) = fs_card(fs_remove(s, item)) + Nat.1
} by {
    if s.contains(item) {
        fs_remove_cardinality_is_suc(s, item)
        s.cardinality_is(fs_card(fs_remove(s, item)) + Nat.1)
        fs_card_eq_of_cardinality_is(s, fs_card(fs_remove(s, item)) + Nat.1)
        fs_card(s) = fs_card(fs_remove(s, item)) + Nat.1
    }
}

/// A finite set containing an element is nonempty.
theorem fs_card_positive_of_contains[T](s: FiniteSet[T], item: T) {
    s.contains(item) implies Nat.0 < fs_card(s)
} by {
    if s.contains(item) {
        let n: Nat = fs_card(fs_remove(s, item))
        fs_card_remove_of_contains(s, item)
        fs_card(s) = n + Nat.1
        add_one_right(n)
        n + Nat.1 = n.suc
        fs_card(s) = n.suc
        alt_suc_ne_zero(n)
        n.suc != Nat.0
        fs_card(s) != Nat.0
        pos_of_ne_zero(fs_card(s))
        Nat.0 < fs_card(s)
    }
}
