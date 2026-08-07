from nat import Nat
from finite_set import FiniteSet
from data.finite.finite_set_card import fs_card, fs_card_cardinality_is, fs_card_eq_of_cardinality_is
from data.finite.finite_set_product import finite_set_product, finite_set_product_cardinality_is

numerals Nat

/// A finite Cartesian product has the product of the cardinalities.
///
/// `finite_set_product_cardinality_is` states this through the `cardinality_is` predicate;
/// counting arguments work with the numeric `fs_card`, so they need this form.
theorem fs_card_product[T, U](a: FiniteSet[T], b: FiniteSet[U]) {
    fs_card(finite_set_product(a, b)) = fs_card(b) * fs_card(a)
} by {
    fs_card_cardinality_is(a)
    a.cardinality_is(fs_card(a))
    fs_card_cardinality_is(b)
    b.cardinality_is(fs_card(b))
    finite_set_product_cardinality_is(a, b, fs_card(a), fs_card(b))
    finite_set_product(a, b).cardinality_is(fs_card(b) * fs_card(a))
    fs_card_eq_of_cardinality_is(finite_set_product(a, b), fs_card(b) * fs_card(a))
    fs_card(finite_set_product(a, b)) = fs_card(b) * fs_card(a)
}
