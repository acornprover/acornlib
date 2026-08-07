from nat import Nat, lte_trans
from finite_set import FiniteSet, fs_union, finite_set_subset_union_left,
    finite_set_subset_union_right
from data.finite.finite_set_filter import finite_set_filter, finite_set_filter_subset
from data.finite.finite_set_card import fs_card
from data.finite.finite_set_card_compare import fs_card_mono

numerals Nat

/// Filtering a finite set can only reduce its count.
theorem fs_card_filter_le[T](s: FiniteSet[T], p: T -> Bool) {
    fs_card(finite_set_filter(s, p)) <= fs_card(s)
} by {
    finite_set_filter_subset(s, p)
    finite_set_filter(s, p).subset_eq(s)
    fs_card_mono(finite_set_filter(s, p), s)
}

/// A union is at least as large as its left part.
theorem fs_card_union_ge_left[T](a: FiniteSet[T], b: FiniteSet[T]) {
    fs_card(a) <= fs_card(fs_union(a, b))
} by {
    finite_set_subset_union_left(a, b)
    a.subset_eq(fs_union(a, b))
    fs_card_mono(a, fs_union(a, b))
}

/// A union is at least as large as its right part.
theorem fs_card_union_ge_right[T](a: FiniteSet[T], b: FiniteSet[T]) {
    fs_card(b) <= fs_card(fs_union(a, b))
} by {
    finite_set_subset_union_right(a, b)
    b.subset_eq(fs_union(a, b))
    fs_card_mono(b, fs_union(a, b))
}

/// A bound on a superset bounds every subset.
theorem fs_card_le_of_subset_le[T](s: FiniteSet[T], t: FiniteSet[T], n: Nat) {
    s.subset_eq(t) and fs_card(t) <= n implies fs_card(s) <= n
} by {
    if s.subset_eq(t) and fs_card(t) <= n {
        fs_card_mono(s, t)
        fs_card(s) <= fs_card(t)
        lte_trans(fs_card(s), fs_card(t), n)
        fs_card(s) <= n
    }
}

/// Counting is antisymmetric on nested sets, so a subset of the same size fills its superset.
theorem fs_card_eq_of_mutual_subset[T](s: FiniteSet[T], t: FiniteSet[T]) {
    s.subset_eq(t) and t.subset_eq(s) implies fs_card(s) = fs_card(t)
} by {
    if s.subset_eq(t) and t.subset_eq(s) {
        fs_card_mono(s, t)
        fs_card(s) <= fs_card(t)
        fs_card_mono(t, s)
        fs_card(t) <= fs_card(s)
        fs_card(s) = fs_card(t)
    }
}
