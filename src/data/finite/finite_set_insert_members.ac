from nat import Nat
from data.basic.set import Set, insert_contains_eq, insert_is_finite_of_finite
from finite_set import FiniteSet, fs_insert, finite_set_ext_contains
from data.finite.finite_set_card import fs_card
from data.finite.finite_set_card_ops import fs_card_insert_of_not_contains

numerals Nat

/// The underlying set of an insertion is the insertion into the underlying set.
///
/// `fs_insert` is characterised by a `FiniteSet.new` equation, so getting at its members needs
/// the constructor unwound once. Everything below is stated in terms of membership instead.
theorem fs_insert_underlying_set[T](s: FiniteSet[T], item: T) {
    fs_insert(s, item).underlying_set = s.underlying_set.insert(item)
} by {
    insert_is_finite_of_finite(s.underlying_set, item)
    s.underlying_set.insert(item).is_finite
    FiniteSet.new(s.underlying_set.insert(item)) = Option.some(fs_insert(s, item))
    fs_insert(s, item).underlying_set = s.underlying_set.insert(item)
}

/// Membership in an insertion.
theorem fs_insert_contains_eq[T](s: FiniteSet[T], item: T, x: T) {
    fs_insert(s, item).contains(x) = (x = item or s.contains(x))
} by {
    fs_insert_underlying_set(s, item)
    (fs_insert(s, item).underlying_set = s.underlying_set.insert(item))
    (fs_insert(s, item).contains(x) = fs_insert(s, item).underlying_set.contains(x))
    (fs_insert(s, item).contains(x) = s.underlying_set.insert(item).contains(x))
    insert_contains_eq(s.underlying_set, item, x)
    (s.underlying_set.insert(item).contains(x) = (x = item or s.underlying_set.contains(x)))
    (s.contains(x) = s.underlying_set.contains(x))
    fs_insert(s, item).contains(x) = (x = item or s.contains(x))
}

/// The inserted element belongs to the insertion.
theorem fs_insert_contains_item[T](s: FiniteSet[T], item: T) {
    fs_insert(s, item).contains(item)
} by {
    fs_insert_contains_eq(s, item, item)
    (fs_insert(s, item).contains(item) = (item = item or s.contains(item)))
    fs_insert(s, item).contains(item)
}

/// A member of the original set belongs to the insertion.
theorem fs_insert_contains_of_contains[T](s: FiniteSet[T], item: T, x: T) {
    s.contains(x) implies fs_insert(s, item).contains(x)
} by {
    if s.contains(x) {
        fs_insert_contains_eq(s, item, x)
        (fs_insert(s, item).contains(x) = (x = item or s.contains(x)))
        fs_insert(s, item).contains(x)
    }
}

/// Inserting an element already present changes nothing.
theorem fs_insert_of_contains[T](s: FiniteSet[T], item: T) {
    s.contains(item) implies fs_insert(s, item) = s
} by {
    if s.contains(item) {
        forall(x: T) {
            fs_insert_contains_eq(s, item, x)
            (fs_insert(s, item).contains(x) = (x = item or s.contains(x)))
            if x = item {
                s.contains(x)
            }
            (fs_insert(s, item).contains(x) = s.contains(x))
            (fs_insert(s, item).contains(x) = fs_insert(s, item).underlying_set.contains(x))
            (s.contains(x) = s.underlying_set.contains(x))
            (fs_insert(s, item).underlying_set.contains(x) = s.underlying_set.contains(x))
        }
        finite_set_ext_contains(fs_insert(s, item), s)
        fs_insert(s, item) = s
    }
}

/// Inserting an element adds at most one to the count.
///
/// The bound rather than the exact count, so that no case analysis on whether the element was
/// already present is needed at the point of use.
theorem fs_card_insert_le[T](s: FiniteSet[T], item: T) {
    fs_card(fs_insert(s, item)) <= fs_card(s) + Nat.1
} by {
    if s.contains(item) {
        fs_insert_of_contains(s, item)
        fs_insert(s, item) = s
        fs_card(fs_insert(s, item)) = fs_card(s)
        (fs_card(s) <= fs_card(s) + Nat.1)
        fs_card(fs_insert(s, item)) <= fs_card(s) + Nat.1
    }
    if not s.contains(item) {
        fs_card_insert_of_not_contains(s, item)
        fs_card(fs_insert(s, item)) = fs_card(s) + Nat.1
        fs_card(fs_insert(s, item)) <= fs_card(s) + Nat.1
    }
    fs_card(fs_insert(s, item)) <= fs_card(s) + Nat.1
}
