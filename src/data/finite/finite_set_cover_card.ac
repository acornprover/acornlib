from nat import Nat, lte_trans
from finite_set import FiniteSet, fs_image, finite_set_maps_into_image
from data.finite.finite_set_card import fs_card
from data.finite.finite_set_card_compare import fs_card_mono
from data.finite.finite_set_image_card import fs_card_image_le
from data.finite.finite_set_subset_intro import fs_subset_eq_intro

numerals Nat

/// True if some member of a finite set is carried to a given value by a map.
///
/// Named rather than left inline inside the covering condition below. An existential nested
/// under a universal is not assembled by proof search, so the covering condition has to be one
/// quantifier deep with this as its body.
define has_preimage_in[T, U](t: FiniteSet[T], f: T -> U, y: U) -> Bool {
    exists(x: T) {
        t.contains(x) and f(x) = y
    }
}

/// A member carried to the value witnesses a preimage.
theorem has_preimage_in_intro[T, U](t: FiniteSet[T], f: T -> U, y: U, x: T) {
    t.contains(x) and f(x) = y implies has_preimage_in(t, f, y)
} by {
    if t.contains(x) and f(x) = y {
        (has_preimage_in(t, f, y) = exists(w: T) {
            t.contains(w) and f(w) = y
        })
        exists(w: T) {
            t.contains(w) and f(w) = y
        }
        has_preimage_in(t, f, y)
    }
}

/// A preimage can be extracted.
theorem has_preimage_in_witness[T, U](t: FiniteSet[T], f: T -> U, y: U) {
    has_preimage_in(t, f, y) implies exists(x: T) {
        t.contains(x) and f(x) = y
    }
} by {
    if has_preimage_in(t, f, y) {
        (has_preimage_in(t, f, y) = exists(w: T) {
            t.contains(w) and f(w) = y
        })
        exists(w: T) {
            t.contains(w) and f(w) = y
        }
    }
}

/// True if every member of a finite set is a value of a map on another finite set.
define is_covered_by_image[T, U](s: FiniteSet[U], t: FiniteSet[T], f: T -> U) -> Bool {
    forall(y: U) {
        s.contains(y) implies has_preimage_in(t, f, y)
    }
}

/// The pointwise covering condition yields the covering predicate.
theorem is_covered_by_image_intro[T, U](s: FiniteSet[U], t: FiniteSet[T], f: T -> U) {
    (forall(y: U) {
        s.contains(y) implies has_preimage_in(t, f, y)
    }) implies is_covered_by_image(s, t, f)
} by {
    if forall(y: U) {
        s.contains(y) implies has_preimage_in(t, f, y)
    } {
        (is_covered_by_image(s, t, f) = forall(y: U) {
            s.contains(y) implies has_preimage_in(t, f, y)
        })
        is_covered_by_image(s, t, f)
    }
}

/// A member of a covered set has a preimage.
theorem is_covered_by_image_apply[T, U](s: FiniteSet[U], t: FiniteSet[T], f: T -> U, y: U) {
    is_covered_by_image(s, t, f) and s.contains(y) implies has_preimage_in(t, f, y)
} by {
    if is_covered_by_image(s, t, f) and s.contains(y) {
        (is_covered_by_image(s, t, f) = forall(z: U) {
            s.contains(z) implies has_preimage_in(t, f, z)
        })
        (s.contains(y) implies has_preimage_in(t, f, y))
        has_preimage_in(t, f, y)
    }
}

/// A covered set is contained in the image.
theorem covered_by_image_subset[T, U](s: FiniteSet[U], t: FiniteSet[T], f: T -> U) {
    is_covered_by_image(s, t, f) implies s.subset_eq(fs_image(t, f))
} by {
    if is_covered_by_image(s, t, f) {
        forall(y: U) {
            if s.contains(y) {
                is_covered_by_image_apply(s, t, f, y)
                has_preimage_in(t, f, y)
                has_preimage_in_witness(t, f, y)
                exists(x: T) {
                    t.contains(x) and f(x) = y
                }
                let (w: T) satisfy {
                    t.contains(w) and f(w) = y
                }
                finite_set_maps_into_image(t, f, w)
                fs_image(t, f).contains(f(w))
                fs_image(t, f).contains(y)
            }
            (s.contains(y) implies fs_image(t, f).contains(y))
        }
        fs_subset_eq_intro(s, fs_image(t, f))
        s.subset_eq(fs_image(t, f))
    }
}

/// A set covered by the values of a map on a finite set is no larger than that set.
///
/// The counting form of a covering argument: to bound a set, exhibit a map from a smaller set
/// whose values reach everything. No injectivity is needed, since a map can identify members
/// but never create new ones.
theorem fs_card_le_of_covered_by_image[T, U](s: FiniteSet[U], t: FiniteSet[T], f: T -> U) {
    is_covered_by_image(s, t, f) implies fs_card(s) <= fs_card(t)
} by {
    if is_covered_by_image(s, t, f) {
        covered_by_image_subset(s, t, f)
        s.subset_eq(fs_image(t, f))
        fs_card_mono(s, fs_image(t, f))
        fs_card(s) <= fs_card(fs_image(t, f))
        fs_card_image_le(t, f)
        fs_card(fs_image(t, f)) <= fs_card(t)
        lte_trans(fs_card(s), fs_card(fs_image(t, f)), fs_card(t))
        fs_card(s) <= fs_card(t)
    }
}
