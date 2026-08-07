from nat import Nat
from finite_set import FiniteSet, finite_set_singleton_cardinality_is_one,
    finite_set_cardinality_is_well_defined
from data.basic.set import Set, cardinality_always_exists, empty_set_cardinality_is_zero

numerals Nat

/// The number of elements of a finite set.
///
/// Every finite set has a cardinality, and that cardinality is unique, so this
/// is well defined. It is the numeric counterpart of the `cardinality_is`
/// predicate, usable wherever a `Nat` is needed.
let fs_card[T](s: FiniteSet[T]) -> result: Nat satisfy {
    s.cardinality_is(result)
} by {
    s.underlying_set.is_finite
    cardinality_always_exists(s.underlying_set)
    let (n: Nat) satisfy {
        s.underlying_set.cardinality_is(n)
    }
    s.cardinality_is(n)
}

/// The cardinality of a finite set agrees with any witness of `cardinality_is`.
theorem fs_card_eq_of_cardinality_is[T](s: FiniteSet[T], n: Nat) {
    s.cardinality_is(n) implies fs_card(s) = n
} by {
    if s.cardinality_is(n) {
        s.cardinality_is(fs_card(s))
        finite_set_cardinality_is_well_defined(s, fs_card(s), n)
    }
}

/// The empty finite set has no elements.
theorem fs_card_empty[T] {
    fs_card(FiniteSet.empty[T]) = Nat.0
} by {
    empty_set_cardinality_is_zero[T]
    FiniteSet.empty[T].underlying_set = Set[T].empty_set
    FiniteSet.empty[T].underlying_set.cardinality_is(Nat.0)
    FiniteSet.empty[T].cardinality_is(Nat.0)
    fs_card_eq_of_cardinality_is(FiniteSet.empty[T], Nat.0)
}

/// A singleton has one element.
theorem fs_card_singleton[T](item: T) {
    fs_card(FiniteSet.empty[T].insert(item)) = Nat.1
} by {
    finite_set_singleton_cardinality_is_one(item)
    FiniteSet.empty[T].insert(item).cardinality_is(Nat.1)
    fs_card_eq_of_cardinality_is(FiniteSet.empty[T].insert(item), Nat.1)
}

/// Finite sets with equal cardinality numbers satisfy the same cardinality predicate.
theorem fs_card_cardinality_is[T](s: FiniteSet[T]) {
    s.cardinality_is(fs_card(s))
}
