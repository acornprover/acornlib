from nat import Nat
from list import List
from data.basic.set import Set, list_set, list_set_contains_eq, insert_contains_eq,
    remove_contains_eq
from data.finite.finite_set_subset_intro import fs_subset_eq_intro
from finite_set import FiniteSet, fs_from_list, fs_insert, fs_remove,
    finite_set_eq_underlying_set, finite_set_ext_contains

numerals Nat

/// The underlying set of a list's finite set is the list's set of members.
theorem fs_from_list_underlying_set[T](items: List[T]) {
    fs_from_list(items).underlying_set = list_set(items)
} by {
    FiniteSet.new(list_set(items)) = Option.some(fs_from_list(items))
}

/// A finite set built from a list contains exactly the list's members.
///
/// The bridge from list membership to finite-set membership. Without it a statement
/// about a finite set cannot be reduced to one about its list representation.
theorem fs_from_list_contains_eq[T](items: List[T], x: T) {
    fs_from_list(items).contains(x) = items.contains(x)
} by {
    fs_from_list_underlying_set(items)
    fs_from_list(items).underlying_set = list_set(items)
    fs_from_list(items).contains(x) = fs_from_list(items).underlying_set.contains(x)
    fs_from_list(items).contains(x) = list_set(items).contains(x)
    list_set_contains_eq(items, x)
    list_set(items).contains(x) = items.contains(x)
}

/// The underlying set of an insertion is the insertion into the underlying set.
theorem fs_insert_underlying_set[T](s: FiniteSet[T], item: T) {
    fs_insert(s, item).underlying_set = s.underlying_set.insert(item)
} by {
    FiniteSet.new(s.underlying_set.insert(item)) = Option.some(fs_insert(s, item))
}

/// Inserting an element adds exactly that element.
///
/// Stated for an arbitrary finite set; the interface previously exposed only the
/// singleton case, which is not enough to reason about a set built up by insertions.
theorem fs_insert_contains_eq[T](s: FiniteSet[T], item: T, x: T) {
    fs_insert(s, item).contains(x) = (x = item or s.contains(x))
} by {
    fs_insert_underlying_set(s, item)
    fs_insert(s, item).underlying_set = s.underlying_set.insert(item)
    fs_insert(s, item).contains(x) = fs_insert(s, item).underlying_set.contains(x)
    fs_insert(s, item).contains(x) = s.underlying_set.insert(item).contains(x)
    insert_contains_eq(s.underlying_set, item, x)
    s.underlying_set.insert(item).contains(x) = (x = item or s.underlying_set.contains(x))
    s.contains(x) = s.underlying_set.contains(x)
}

/// An inserted element belongs to the result.
theorem fs_insert_contains_self[T](s: FiniteSet[T], item: T) {
    fs_insert(s, item).contains(item)
} by {
    fs_insert_contains_eq(s, item, item)
    fs_insert(s, item).contains(item) = (item = item or s.contains(item))
}

/// Members of the original set survive an insertion.
theorem fs_insert_contains_of_contains[T](s: FiniteSet[T], item: T, x: T) {
    s.contains(x) implies fs_insert(s, item).contains(x)
} by {
    if s.contains(x) {
        fs_insert_contains_eq(s, item, x)
        fs_insert(s, item).contains(x) = (x = item or s.contains(x))
        fs_insert(s, item).contains(x)
    }
}

/// A member of an insertion is either the inserted element or an original member.
theorem fs_insert_contains_cases[T](s: FiniteSet[T], item: T, x: T) {
    fs_insert(s, item).contains(x) implies x = item or s.contains(x)
} by {
    fs_insert_contains_eq(s, item, x)
}

/// Membership in a finite set with one element removed.
///
/// `src/finite_set/` defines `fs_remove` through the underlying set but states the membership
/// rule only at the `Set` level, so this lifts it, as `fs_insert_contains_eq` does for
/// insertion.
theorem fs_remove_contains_eq[T](s: FiniteSet[T], item: T, x: T) {
    fs_remove(s, item).contains(x) = (s.contains(x) and x != item)
} by {
    fs_remove(s, item).underlying_set = s.underlying_set.remove(item)
    fs_remove(s, item).contains(x) = fs_remove(s, item).underlying_set.contains(x)
    fs_remove(s, item).contains(x) = s.underlying_set.remove(item).contains(x)
    remove_contains_eq(s.underlying_set, item, x)
    (s.underlying_set.remove(item).contains(x)
        = (s.underlying_set.contains(x) and x != item))
    s.contains(x) = s.underlying_set.contains(x)
}

/// Removing an element leaves a subset.
theorem fs_remove_subset[T](s: FiniteSet[T], item: T) {
    fs_remove(s, item).subset_eq(s)
} by {
    forall(x: T) {
        if fs_remove(s, item).contains(x) {
            fs_remove_contains_eq(s, item, x)
            s.contains(x) and x != item
            s.contains(x)
        }
        (fs_remove(s, item).contains(x) implies s.contains(x))
    }
    fs_subset_eq_intro(fs_remove(s, item), s)
    fs_remove(s, item).subset_eq(s)
}

/// Two finite sets with the same members are equal.
///
/// `finite_set_ext_contains` is stated over `underlying_set`, so using it means carrying
/// the membership equality across the `contains` definition at every call site. This states
/// it directly in terms of `contains`, which is how callers actually have their facts.
theorem finite_set_eq_of_contains_eq[T](a: FiniteSet[T], b: FiniteSet[T]) {
    (forall(x: T) { a.contains(x) = b.contains(x) }) implies a = b
} by {
    if forall(x: T) { a.contains(x) = b.contains(x) } {
        forall(x: T) {
            a.contains(x) = b.contains(x)
            a.contains(x) = a.underlying_set.contains(x)
            b.contains(x) = b.underlying_set.contains(x)
            a.underlying_set.contains(x) = b.underlying_set.contains(x)
        }
        finite_set_ext_contains(a, b)
        a = b
    }
}
