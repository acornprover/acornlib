from nat import Nat
from list import List, unique_implies_tail_unique
from data.list.list_unique_cons import unique_cons_head_not_in_tail
from finite_set import FiniteSet, fs_from_list, fs_insert
from data.finite.finite_set_induction import finite_set_has_unique_list
from data.finite.finite_set_membership import fs_from_list_contains_eq
from data.finite.finite_set_from_list_structure import fs_from_list_nil, fs_from_list_cons

numerals Nat

/// A property preserved by inserting a new element holds of the finite set of any list
/// without repeats.
///
/// The stronger companion to the plain induction principle: the step may assume the
/// inserted element is not already present. That assumption is what a counting argument
/// needs, and it is available here because a list without repeats does not contain its own
/// head.
theorem finite_set_unique_list_property[T](p: FiniteSet[T] -> Bool, items: List[T]) {
    p(FiniteSet.empty[T]) and (forall(s: FiniteSet[T], item: T) {
        p(s) and not s.contains(item) implies p(fs_insert(s, item))
    }) and items.is_unique implies p(fs_from_list(items))
} by {
    define q(x: List[T]) -> Bool {
        p(FiniteSet.empty[T]) and (forall(s: FiniteSet[T], item: T) {
            p(s) and not s.contains(item) implies p(fs_insert(s, item))
        }) and x.is_unique implies p(fs_from_list(x))
    }
    q(List.nil[T])
    forall(head: T, tail: List[T]) {
        if q(tail) {
            if p(FiniteSet.empty[T]) and (forall(s: FiniteSet[T], item: T) {
                p(s) and not s.contains(item) implies p(fs_insert(s, item))
            }) and List.cons(head, tail).is_unique {
                unique_implies_tail_unique(head, tail)
                tail.is_unique
                p(fs_from_list(tail))
                unique_cons_head_not_in_tail(head, tail)
                not tail.contains(head)
                fs_from_list_contains_eq(tail, head)
                fs_from_list(tail).contains(head) = tail.contains(head)
                not fs_from_list(tail).contains(head)
                p(fs_from_list(tail)) and not fs_from_list(tail).contains(head) implies p(fs_insert(fs_from_list(tail), head))
                p(fs_insert(fs_from_list(tail), head))
                fs_from_list_cons(head, tail)
                fs_from_list(List.cons(head, tail)) = fs_insert(fs_from_list(tail), head)
                p(fs_from_list(List.cons(head, tail)))
            }
            q(List.cons(head, tail))
        }
    }
}

/// Induction on finite sets, with the step assuming the inserted element is new.
///
/// This is the form a counting argument needs, since it may assume each element is added
/// exactly once. It rests on the fact that a list without repeats excludes its own head.
theorem finite_set_strong_induction[T](p: FiniteSet[T] -> Bool, s: FiniteSet[T]) {
    p(FiniteSet.empty[T]) and (forall(t: FiniteSet[T], item: T) {
        p(t) and not t.contains(item) implies p(fs_insert(t, item))
    }) implies p(s)
} by {
    if p(FiniteSet.empty[T]) and forall(t: FiniteSet[T], item: T) {
        p(t) and not t.contains(item) implies p(fs_insert(t, item))
    } {
        finite_set_has_unique_list(s)
        let (rep: List[T]) satisfy {
            fs_from_list(rep) = s and rep.is_unique
        }
        finite_set_unique_list_property(p, rep)
        p(fs_from_list(rep))
        fs_from_list(rep) = s
        p(s)
    }
}
