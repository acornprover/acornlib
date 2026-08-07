from nat import Nat
from data.basic.set import Set, subset_contains_eq
from finite_set import FiniteSet, fs_intersection, fs_difference, fs_union,
    finite_set_intersection_contains_eq, finite_set_difference_contains_eq,
    finite_set_union_contains_eq, finite_set_subset_contains
from data.finite.finite_set_card import fs_card
from data.finite.finite_set_card_compare import fs_card_mono

numerals Nat

/// A pointwise containment condition makes one finite set a subset of another.
///
/// `subset_eq` unfolds to `Set.subset`, which `subset_contains_eq` characterizes
/// pointwise. This is the introduction lemma that lets new subset facts be proved
/// from membership reasoning.
theorem fs_subset_eq_intro[T](a: FiniteSet[T], b: FiniteSet[T]) {
    (forall(x: T) { a.contains(x) implies b.contains(x) }) implies a.subset_eq(b)
} by {
    if forall(x: T) { a.contains(x) implies b.contains(x) } {
        forall(x: T) {
            a.contains(x) = a.underlying_set.contains(x)
            b.contains(x) = b.underlying_set.contains(x)
            a.underlying_set.contains(x) implies b.underlying_set.contains(x)
        }
        subset_contains_eq(a.underlying_set, b.underlying_set)
        a.underlying_set.subset(b.underlying_set) = forall(x: T) {
            a.underlying_set.contains(x) implies b.underlying_set.contains(x)
        }
        a.underlying_set.subset(b.underlying_set)
        a.subset_eq(b)
    }
}

/// An intersection is contained in its left part.
theorem fs_intersection_subset_left[T](a: FiniteSet[T], b: FiniteSet[T]) {
    fs_intersection(a, b).subset_eq(a)
} by {
    forall(x: T) {
        if fs_intersection(a, b).contains(x) {
            finite_set_intersection_contains_eq(a, b, x)
            fs_intersection(a, b).contains(x) = (a.contains(x) and b.contains(x))
            a.contains(x)
        }
        fs_intersection(a, b).contains(x) implies a.contains(x)
    }
    fs_subset_eq_intro(fs_intersection(a, b), a)
    fs_intersection(a, b).subset_eq(a)
}

/// An intersection is contained in its right part.
theorem fs_intersection_subset_right[T](a: FiniteSet[T], b: FiniteSet[T]) {
    fs_intersection(a, b).subset_eq(b)
} by {
    forall(x: T) {
        if fs_intersection(a, b).contains(x) {
            finite_set_intersection_contains_eq(a, b, x)
            fs_intersection(a, b).contains(x) = (a.contains(x) and b.contains(x))
            b.contains(x)
        }
        fs_intersection(a, b).contains(x) implies b.contains(x)
    }
    fs_subset_eq_intro(fs_intersection(a, b), b)
    fs_intersection(a, b).subset_eq(b)
}

/// A difference is contained in the set it is taken from.
theorem fs_difference_subset[T](a: FiniteSet[T], b: FiniteSet[T]) {
    fs_difference(a, b).subset_eq(a)
} by {
    forall(x: T) {
        if fs_difference(a, b).contains(x) {
            finite_set_difference_contains_eq(a, b, x)
            fs_difference(a, b).contains(x) = (a.contains(x) and not b.contains(x))
            a.contains(x)
        }
        fs_difference(a, b).contains(x) implies a.contains(x)
    }
    fs_subset_eq_intro(fs_difference(a, b), a)
    fs_difference(a, b).subset_eq(a)
}

/// Intersecting can only reduce a count, measured against the left part.
theorem fs_card_intersection_le_left[T](a: FiniteSet[T], b: FiniteSet[T]) {
    fs_card(fs_intersection(a, b)) <= fs_card(a)
} by {
    fs_intersection_subset_left(a, b)
    fs_card_mono(fs_intersection(a, b), a)
}

/// Intersecting can only reduce a count, measured against the right part.
theorem fs_card_intersection_le_right[T](a: FiniteSet[T], b: FiniteSet[T]) {
    fs_card(fs_intersection(a, b)) <= fs_card(b)
} by {
    fs_intersection_subset_right(a, b)
    fs_card_mono(fs_intersection(a, b), b)
}

/// Removing elements can only reduce a count.
theorem fs_card_difference_le[T](a: FiniteSet[T], b: FiniteSet[T]) {
    fs_card(fs_difference(a, b)) <= fs_card(a)
} by {
    fs_difference_subset(a, b)
    fs_card_mono(fs_difference(a, b), a)
}

/// A set is contained in its union with anything.
theorem fs_subset_union_of_contains[T](a: FiniteSet[T], b: FiniteSet[T], c: FiniteSet[T]) {
    a.subset_eq(c) and b.subset_eq(c) implies fs_union(a, b).subset_eq(c)
} by {
    if a.subset_eq(c) and b.subset_eq(c) {
        forall(x: T) {
            if fs_union(a, b).contains(x) {
                finite_set_union_contains_eq(a, b, x)
                fs_union(a, b).contains(x) = (a.contains(x) or b.contains(x))
                if a.contains(x) {
                    finite_set_subset_contains(a, c, x)
                    c.contains(x)
                }
                if not a.contains(x) {
                    b.contains(x)
                    finite_set_subset_contains(b, c, x)
                    c.contains(x)
                }
                c.contains(x)
            }
            fs_union(a, b).contains(x) implies c.contains(x)
        }
        fs_subset_eq_intro(fs_union(a, b), c)
        fs_union(a, b).subset_eq(c)
    }
}

/// A union of two sets inside a common bound is itself bounded by that count.
theorem fs_card_union_le_of_bounds[T](
    a: FiniteSet[T], b: FiniteSet[T], c: FiniteSet[T]
) {
    a.subset_eq(c) and b.subset_eq(c) implies fs_card(fs_union(a, b)) <= fs_card(c)
} by {
    if a.subset_eq(c) and b.subset_eq(c) {
        fs_subset_union_of_contains(a, b, c)
        fs_union(a, b).subset_eq(c)
        fs_card_mono(fs_union(a, b), c)
    }
}
