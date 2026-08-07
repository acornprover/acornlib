from nat import Nat, pos_of_ne_zero
from finite_set import FiniteSet, fs_insert
from data.finite.finite_set_card import fs_card, fs_card_empty
from data.finite.finite_set_membership import fs_insert_contains_self
from data.finite.finite_set_induction_principle import finite_set_induction_apply

numerals Nat

/// True of a finite set that is empty or has a member.
///
/// Phrased as a disjunction so the empty case is a real statement rather than a
/// vacuous one, which Acorn rejects.
define is_empty_or_inhabited[T](s: FiniteSet[T]) -> Bool {
    s = FiniteSet.empty[T] or exists(x: T) { s.contains(x) }
}

/// The empty set satisfies the disjunction on the left.
theorem is_empty_or_inhabited_empty[T] {
    is_empty_or_inhabited(FiniteSet.empty[T])
} by {
    FiniteSet.empty[T] = FiniteSet.empty[T]
    is_empty_or_inhabited(FiniteSet.empty[T])
}

/// Inserting an element gives a set with a member.
theorem is_empty_or_inhabited_insert[T](s: FiniteSet[T], item: T) {
    is_empty_or_inhabited(s) implies is_empty_or_inhabited(fs_insert(s, item))
} by {
    if is_empty_or_inhabited(s) {
        fs_insert_contains_self(s, item)
        fs_insert(s, item).contains(item)
        exists(x: T) { fs_insert(s, item).contains(x) }
        is_empty_or_inhabited(fs_insert(s, item))
    }
}

/// Every finite set is empty or has a member.
///
/// The first use of finite-set induction. Without it there was no way to get from a
/// nonempty set to one of its elements, which is why earlier work had to carry a
/// witnessing function instead.
theorem finite_set_empty_or_inhabited[T](s: FiniteSet[T]) {
    is_empty_or_inhabited(s)
} by {
    is_empty_or_inhabited_empty[T]
    forall(t: FiniteSet[T], item: T) {
        is_empty_or_inhabited_insert(t, item)
        is_empty_or_inhabited(t) implies is_empty_or_inhabited(fs_insert(t, item))
    }
    finite_set_induction_apply(is_empty_or_inhabited[T], s)
    is_empty_or_inhabited(s)
}

/// A finite set other than the empty one has a member.
theorem finite_set_has_member_of_ne_empty[T](s: FiniteSet[T]) {
    s != FiniteSet.empty[T] implies exists(x: T) { s.contains(x) }
} by {
    if s != FiniteSet.empty[T] {
        finite_set_empty_or_inhabited(s)
        is_empty_or_inhabited(s)
        s = FiniteSet.empty[T] or exists(x: T) { s.contains(x) }
        exists(x: T) { s.contains(x) }
    }
}

/// A finite set with a positive count has a member.
///
/// This is the extraction that lets a positive degree yield an actual neighbor.
theorem finite_set_has_member_of_positive_card[T](s: FiniteSet[T]) {
    Nat.0 < fs_card(s) implies exists(x: T) { s.contains(x) }
} by {
    if Nat.0 < fs_card(s) {
        if s = FiniteSet.empty[T] {
            fs_card_empty[T]
            fs_card(FiniteSet.empty[T]) = Nat.0
            fs_card(s) = Nat.0
            false
        }
        s != FiniteSet.empty[T]
        finite_set_has_member_of_ne_empty(s)
        exists(x: T) { s.contains(x) }
    }
}
