from finite_set import FiniteSet, fs_from_list, finite_set_has_unique_list
from finite_set import finite_set_sum, finite_set_sum_eq_list_sum
from list import List, map, length_range, sum, partial, map_length, map_range,
    list_extensionality, map_under_idx
from real import Real
from real import partial_seq_lte, seq_lte, is_lower_bound, partial_nonneg
from algebra.add_ordered_group import add_le_add
from nat import Nat

numerals Nat

/// True if `f` is pointwise less than or equal to `g` everywhere.
define pointwise_le[T](f: T -> Real, g: T -> Real) -> Bool {
    forall(t: T) { f(t) <= g(t) }
}

/// Instantiating a pointwise inequality at a single point.
theorem pointwise_le_at[T](f: T -> Real, g: T -> Real, t: T) {
    pointwise_le(f, g) implies f(t) <= g(t)
} by {
    if pointwise_le(f, g) {
        f(t) <= g(t)
    }
}

/// Cons step for the pointwise sum inequality: if the inequality holds for the tail
/// and the head, it holds for the consed list.
theorem list_map_sum_le_cons[T](head: T, tail: List[T], f: T -> Real, g: T -> Real) {
    sum[Real](map(tail, f)) <= sum[Real](map(tail, g)) and f(head) <= g(head)
        implies sum[Real](map(List.cons(head, tail), f)) <= sum[Real](map(List.cons(head, tail), g))
} by {
    if sum[Real](map(tail, f)) <= sum[Real](map(tail, g)) and f(head) <= g(head) {
        map[T, Real](List.cons(head, tail), g) = List.cons(g(head), map(tail, g))
        sum[Real](List.cons(f(head), map(tail, f))) = f(head) + sum[Real](map(tail, f))
        sum[Real](List.cons(g(head), map(tail, g))) = g(head) + sum[Real](map(tail, g))
        sum[Real](map(List.cons(head, tail), f)) = f(head) + sum[Real](map(tail, f))
        sum[Real](map(List.cons(head, tail), g)) = g(head) + sum[Real](map(tail, g))
        add_le_add[Real](f(head), g(head), sum[Real](map(tail, f)), sum[Real](map(tail, g)))
        f(head) + sum[Real](map(tail, f)) <= g(head) + sum[Real](map(tail, g))
        sum[Real](map(List.cons(head, tail), f)) <= sum[Real](map(List.cons(head, tail), g))
    }
}

/// Pointwise ordered Nat-indexed real sequences have ordered sums over `n.range`.
lemma range_map_sum_le(f: Nat -> Real, g: Nat -> Real, n: Nat) {
    seq_lte(f, g) implies sum[Real](map(n.range, f)) <= sum[Real](map(n.range, g))
} by {
    if seq_lte(f, g) {
        partial_seq_lte(f, g)
        seq_lte(partial[Real](f), partial[Real](g))
        seq_lte(partial[Real](f), partial[Real](g)) = forall(k: Nat) {
            partial[Real](f, k) <= partial[Real](g, k)
        }
        partial[Real](f, n) <= partial[Real](g, n)
        partial[Real](f, n) = sum[Real](map(n.range, f))
        partial[Real](g, n) = sum[Real](map(n.range, g))
        sum[Real](map(n.range, f)) <= sum[Real](map(n.range, g))
    }
}

/// Nonnegative Nat-indexed real sequences have nonnegative sums over `n.range`.
lemma range_map_sum_nonneg(f: Nat -> Real, n: Nat) {
    is_lower_bound(f, Real.0) implies Real.0 <= sum[Real](map(n.range, f))
} by {
    if is_lower_bound(f, Real.0) {
        partial_nonneg(f, n)
        partial[Real](f, n) >= Real.0
        partial[Real](f, n) = sum[Real](map(n.range, f))
        Real.0 <= sum[Real](map(n.range, f))
    }
}

/// The Nat-indexed sequence whose first entries are `map(items, f)` and whose
/// later entries are zero.
define list_map_term[T](items: List[T], f: T -> Real, index: Nat) -> Real {
    match items.get_idx(index) {
        Option.none {
            Real.0
        }
        Option.some(item) {
            f(item)
        }
    }
}

/// Mapping a real-valued function over a list agrees with mapping its indexed
/// sequence over the range of valid indices.
lemma list_map_eq_range_map_term[T](items: List[T], f: T -> Real) {
    map(items, f) = map(items.length.range, list_map_term(items, f))
} by {
    let lhs = map(items, f)
    let rhs = map(items.length.range, list_map_term(items, f))
    map_length(items, f)
    lhs.length = items.length
    length_range(items.length)
    items.length.range.length = items.length
    map_length(items.length.range, list_map_term(items, f))
    rhs.length = items.length.range.length
    rhs.length = items.length
    lhs.length = rhs.length

    forall(i: Nat) {
        if i < lhs.length {
            i < items.length
            map_range[Real](items.length, i, list_map_term(items, f))
            rhs.get_idx(i) = Option.some(list_map_term(items, f, i))
            match items.get_idx(i) {
                Option.none {
                    list_map_term(items, f, i) = Real.0
                    map_under_idx(items, f, i)
                    exists(x: T) {
                        Option.some(x) = items.get_idx(i) and map(items, f).get_idx(i) = Option.some(f(x))
                    }
                    let x: T satisfy {
                        Option.some(x) = items.get_idx(i) and map(items, f).get_idx(i) = Option.some(f(x))
                    }
                    Option.some(x) = Option.none[T]
                    false
                }
                Option.some(item) {
                    list_map_term(items, f, i) = f(item)
                    map_under_idx(items, f, i)
                    exists(x: T) {
                        Option.some(x) = items.get_idx(i) and map(items, f).get_idx(i) = Option.some(f(x))
                    }
                    let x: T satisfy {
                        Option.some(x) = items.get_idx(i) and map(items, f).get_idx(i) = Option.some(f(x))
                    }
                    Option.some(x) = Option.some(item)
                    x = item
                    lhs.get_idx(i) = map(items, f).get_idx(i)
                    map(items, f).get_idx(i) = Option.some(f(item))
                    lhs.get_idx(i) = Option.some(f(item))
                    rhs.get_idx(i) = Option.some(f(item))
                    lhs.get_idx(i) = rhs.get_idx(i)
                }
            }
        }
    }
    forall(i: Nat) { i < lhs.length implies lhs.get_idx(i) = rhs.get_idx(i) }
    lhs.length = rhs.length and (forall(i: Nat) { i < lhs.length implies lhs.get_idx(i) = rhs.get_idx(i) })
    list_extensionality(lhs, rhs)
    lhs = rhs
}

/// The list sum of a real-valued map is the sum of its indexed sequence over
/// the range of valid indices.
lemma list_map_sum_eq_range_map_sum[T](items: List[T], f: T -> Real) {
    sum[Real](map(items, f)) = sum[Real](map(items.length.range, list_map_term(items, f)))
} by {
    list_map_eq_range_map_term(items, f)
}

/// Pointwise ordered real-valued functions have ordered sums over any list.
theorem list_map_sum_le[T](items: List[T], f: T -> Real, g: T -> Real) {
    pointwise_le(f, g) implies sum[Real](map(items, f)) <= sum[Real](map(items, g))
} by {
    if pointwise_le(f, g) {
        let f_indexed = list_map_term(items, f)
        let g_indexed = list_map_term(items, g)
        forall(index: Nat) {
            match items.get_idx(index) {
                Option.none {
                    f_indexed(index) = Real.0
                    g_indexed(index) = Real.0
                    Real.0 <= Real.0
                    f_indexed(index) <= g_indexed(index)
                }
                Option.some(item) {
                    f_indexed(index) = f(item)
                    g_indexed(index) = g(item)
                    pointwise_le_at(f, g, item)
                    f(item) <= g(item)
                    f_indexed(index) <= g_indexed(index)
                }
            }
        }
        seq_lte(f_indexed, g_indexed) = forall(index: Nat) {
            f_indexed(index) <= g_indexed(index)
        }
        seq_lte(f_indexed, g_indexed)
        range_map_sum_le(f_indexed, g_indexed, items.length)
        sum[Real](map(items.length.range, f_indexed)) <= sum[Real](map(items.length.range, g_indexed))
        list_map_sum_eq_range_map_sum(items, f)
        list_map_sum_eq_range_map_sum(items, g)
        sum[Real](map(items, f)) <= sum[Real](map(items, g))
    }
}

/// Lists of nonnegative real values have nonnegative sums.
lemma list_map_sum_nonneg[T](items: List[T], f: T -> Real) {
    (forall(item: T) { Real.0 <= f(item) }) implies Real.0 <= sum[Real](map(items, f))
} by {
    if forall(item: T) { Real.0 <= f(item) } {
        let f_indexed = list_map_term(items, f)
        forall(index: Nat) {
            match items.get_idx(index) {
                Option.none {
                    f_indexed(index) = Real.0
                    Real.0 <= f_indexed(index)
                }
                Option.some(item) {
                    f_indexed(index) = f(item)
                    Real.0 <= f(item)
                    Real.0 <= f_indexed(index)
                }
            }
        }
        is_lower_bound(f_indexed, Real.0) = forall(index: Nat) {
            Real.0 <= f_indexed(index)
        }
        is_lower_bound(f_indexed, Real.0)
        range_map_sum_nonneg(f_indexed, items.length)
        Real.0 <= sum[Real](map(items.length.range, f_indexed))
        list_map_sum_eq_range_map_sum(items, f)
        Real.0 <= sum[Real](map(items, f))
    }
}

/// Pointwise ordered real-valued functions have ordered finite-set sums.
theorem finite_set_sum_le[T](s: FiniteSet[T], f: T -> Real, g: T -> Real) {
    pointwise_le(f, g) implies finite_set_sum(s, f) <= finite_set_sum(s, g)
} by {
    if pointwise_le(f, g) {
        finite_set_has_unique_list(s)
        let items: List[T] satisfy {
            fs_from_list(items) = s and items.is_unique
        }
        finite_set_sum_eq_list_sum[T, Real](items, f)
        finite_set_sum(fs_from_list(items), f) = sum[Real](map(items, f))
        finite_set_sum_eq_list_sum[T, Real](items, g)
        finite_set_sum(fs_from_list(items), g) = sum[Real](map(items, g))
        list_map_sum_le(items, f, g)
        sum[Real](map(items, f)) <= sum[Real](map(items, g))
        finite_set_sum(s, f) <= finite_set_sum(s, g)
    }
}

/// Finite-set sums of nonnegative real-valued functions are nonnegative.
theorem finite_set_sum_nonneg[T](s: FiniteSet[T], f: T -> Real) {
    (forall(item: T) { Real.0 <= f(item) }) implies Real.0 <= finite_set_sum(s, f)
} by {
    if forall(item: T) { Real.0 <= f(item) } {
        finite_set_has_unique_list(s)
        let items: List[T] satisfy {
            fs_from_list(items) = s and items.is_unique
        }
        finite_set_sum_eq_list_sum[T, Real](items, f)
        finite_set_sum(fs_from_list(items), f) = sum[Real](map(items, f))
        list_map_sum_nonneg(items, f)
        Real.0 <= sum[Real](map(items, f))
        Real.0 <= finite_set_sum(s, f)
    }
}
