from nat import Nat
from list import List
from finite_set import FiniteSet, fs_from_list, fs_insert
from data.finite.finite_set_induction import finite_set_has_unique_list
from data.finite.finite_set_from_list_structure import fs_from_list_nil, fs_from_list_cons

numerals Nat

/// A property preserved by insertion holds of the finite set of any list.
///
/// The list is a parameter rather than a bound variable, so the goal has the same shape as
/// the induction predicate and structural induction discharges it directly. This is the
/// form the list library's own induction proofs use.
theorem finite_set_list_rep_property[T](p: FiniteSet[T] -> Bool, items: List[T]) {
    p(FiniteSet.empty[T]) and (forall(s: FiniteSet[T], item: T) {
        p(s) implies p(fs_insert(s, item))
    }) implies p(fs_from_list(items))
} by {
    define q(x: List[T]) -> Bool {
        p(FiniteSet.empty[T]) and (forall(s: FiniteSet[T], item: T) {
            p(s) implies p(fs_insert(s, item))
        }) implies p(fs_from_list(x))
    }
    q(List.nil[T])
    forall(head: T, tail: List[T]) {
        if q(tail) {
            if p(FiniteSet.empty[T]) and forall(s: FiniteSet[T], item: T) {
                p(s) implies p(fs_insert(s, item))
            } {
                p(fs_from_list(tail))
                p(fs_from_list(tail)) implies p(fs_insert(fs_from_list(tail), head))
                p(fs_insert(fs_from_list(tail), head))
                fs_from_list_cons(head, tail)
                fs_from_list(List.cons(head, tail)) = fs_insert(fs_from_list(tail), head)
                p(fs_from_list(List.cons(head, tail)))
            }
            q(List.cons(head, tail))
        }
    }
}

/// Induction on finite sets.
///
/// A property that holds of the empty set and survives inserting any element holds of
/// every finite set.
///
/// The step hypothesis does not assume the inserted element is new. That is what lets the
/// proof run over an arbitrary list representation rather than a repeat-free one,
/// sidestepping the fact that `is_unique` is not a structural property.
theorem finite_set_induction[T](p: FiniteSet[T] -> Bool) {
    p(FiniteSet.empty[T]) and (forall(s: FiniteSet[T], item: T) {
        p(s) implies p(fs_insert(s, item))
    }) implies forall(s: FiniteSet[T]) { p(s) }
} by {
    if p(FiniteSet.empty[T]) and forall(s: FiniteSet[T], item: T) {
        p(s) implies p(fs_insert(s, item))
    } {
        forall(s: FiniteSet[T]) {
            finite_set_has_unique_list(s)
            let (rep: List[T]) satisfy {
                fs_from_list(rep) = s and rep.is_unique
            }
            finite_set_list_rep_property(p, rep)
            p(fs_from_list(rep))
            fs_from_list(rep) = s
            p(s)
        }
    }
}

/// Induction on finite sets, applied to a specific set.
theorem finite_set_induction_apply[T](p: FiniteSet[T] -> Bool, s: FiniteSet[T]) {
    p(FiniteSet.empty[T]) and (forall(t: FiniteSet[T], item: T) {
        p(t) implies p(fs_insert(t, item))
    }) implies p(s)
} by {
    if p(FiniteSet.empty[T]) and forall(t: FiniteSet[T], item: T) {
        p(t) implies p(fs_insert(t, item))
    } {
        finite_set_induction(p)
        forall(t: FiniteSet[T]) { p(t) }
        p(s)
    }
}
