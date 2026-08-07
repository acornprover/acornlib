from nat import Nat, only_zero_lte_zero, lte_cancel_suc, lte_and_lt, lt_and_lte
from list import List
from finite_set import FiniteSet, fs_from_list, fs_insert,
    finite_set_cardinality_has_exact_unique_list, finite_set_singleton_subset_of_contains,
    finite_set_empty_contains_eq
from data.finite.finite_set_card import fs_card, fs_card_cardinality_is, fs_card_singleton,
    fs_card_empty
from data.finite.finite_set_card_ops import fs_card_insert_of_not_contains
from data.finite.finite_set_subset_intro import fs_subset_eq_intro
from data.finite.finite_set_card_compare import fs_card_mono
from data.finite.finite_set_membership import fs_from_list_contains_eq, fs_insert_contains_eq
from data.list.list_unique_cons import unique_cons_head_not_in_tail

numerals Nat

/// A finite set with at least two elements has two distinct members.
///
/// Read off a duplicate-free list representation: a list of length at least two is a cons of a
/// cons, and uniqueness makes its first two entries different. Counting arguments that need to
/// produce a second witness go through this.
theorem fs_two_distinct_members[T](s: FiniteSet[T]) {
    Nat.2 <= fs_card(s) implies exists(a: T, b: T) {
        s.contains(a) and s.contains(b) and a != b
    }
} by {
    if Nat.2 <= fs_card(s) {
        fs_card_cardinality_is(s)
        s.cardinality_is(fs_card(s))
        finite_set_cardinality_has_exact_unique_list(s, fs_card(s))
        exists(items: List[T]) {
            fs_from_list(items) = s and items.is_unique and items.length = fs_card(s)
        }
        let (items: List[T]) satisfy {
            fs_from_list(items) = s and items.is_unique and items.length = fs_card(s)
        }
        match items {
            List.nil {
                List.nil[T].length = Nat.0
                Nat.2 <= Nat.0
                only_zero_lte_zero(Nat.2)
                Nat.2 = Nat.0
                false
            }
            List.cons(head, tail) {
                List.cons(head, tail).length = tail.length + Nat.1
                match tail {
                    List.nil {
                        List.nil[T].length = Nat.0
                        List.cons(head, List.nil[T]).length = Nat.1
                        Nat.2 <= Nat.1
                        Nat.1 < Nat.2
                        lte_and_lt(Nat.2, Nat.1, Nat.2)
                        Nat.2 < Nat.2
                        false
                    }
                    List.cons(second, rest) {
                        List.cons(head, List.cons(second, rest)).is_unique
                        unique_cons_head_not_in_tail(head, List.cons(second, rest))
                        not List.cons(second, rest).contains(head)
                        List.cons(second, rest).contains(second)
                        head != second
                        List.cons(head, List.cons(second, rest)).contains(head)
                        items.contains(head)
                        List.cons(head, List.cons(second, rest)).contains(second)
                        items.contains(second)
                        fs_from_list_contains_eq(items, head)
                        fs_from_list(items).contains(head)
                        s.contains(head)
                        fs_from_list_contains_eq(items, second)
                        fs_from_list(items).contains(second)
                        s.contains(second)
                        exists(a: T, b: T) {
                            s.contains(a) and s.contains(b) and a != b
                        }
                    }
                }
            }
        }
    }
}

/// A finite set with a member is not empty by count.
///
/// The one-element subset it contains has cardinality one, and cardinality is monotone. This
/// is the form counting arguments need: they have a member and want a numeric lower bound.
theorem fs_card_pos_of_contains[T](s: FiniteSet[T], x: T) {
    s.contains(x) implies Nat.1 <= fs_card(s)
} by {
    if s.contains(x) {
        finite_set_singleton_subset_of_contains(s, x)
        fs_insert(FiniteSet.empty[T], x).subset_eq(s)
        fs_card_mono(fs_insert(FiniteSet.empty[T], x), s)
        fs_card(fs_insert(FiniteSet.empty[T], x)) <= fs_card(s)
        fs_card_singleton(x)
        fs_card(fs_insert(FiniteSet.empty[T], x)) = Nat.1
        Nat.1 <= fs_card(s)
    }
}

/// Two distinct members force a cardinality of at least two.
///
/// The converse of `fs_two_distinct_members`: that reads two members off a count, this reads a
/// count off two members. Counting arguments that rule out a singleton need this direction.
theorem fs_card_two_of_distinct_members[T](s: FiniteSet[T], x: T, y: T) {
    s.contains(x) and s.contains(y) and x != y implies Nat.2 <= fs_card(s)
} by {
    if s.contains(x) and s.contains(y) and x != y {
        finite_set_empty_contains_eq(y)
        not FiniteSet.empty[T].contains(y)
        fs_card_insert_of_not_contains(FiniteSet.empty[T], y)
        (fs_card(fs_insert(FiniteSet.empty[T], y))
            = fs_card(FiniteSet.empty[T]) + Nat.1)
        fs_card_empty[T]
        fs_card(FiniteSet.empty[T]) = Nat.0
        fs_card(fs_insert(FiniteSet.empty[T], y)) = Nat.1
        fs_insert_contains_eq(FiniteSet.empty[T], y, x)
        (fs_insert(FiniteSet.empty[T], y).contains(x)
            = (x = y or FiniteSet.empty[T].contains(x)))
        finite_set_empty_contains_eq(x)
        not fs_insert(FiniteSet.empty[T], y).contains(x)
        fs_card_insert_of_not_contains(fs_insert(FiniteSet.empty[T], y), x)
        (fs_card(fs_insert(fs_insert(FiniteSet.empty[T], y), x))
            = fs_card(fs_insert(FiniteSet.empty[T], y)) + Nat.1)
        fs_card(fs_insert(fs_insert(FiniteSet.empty[T], y), x)) = Nat.2
        forall(z: T) {
            if fs_insert(fs_insert(FiniteSet.empty[T], y), x).contains(z) {
                fs_insert_contains_eq(fs_insert(FiniteSet.empty[T], y), x, z)
                (fs_insert(fs_insert(FiniteSet.empty[T], y), x).contains(z)
                    = (z = x or fs_insert(FiniteSet.empty[T], y).contains(z)))
                if z = x {
                    s.contains(z)
                }
                if z != x {
                    fs_insert(FiniteSet.empty[T], y).contains(z)
                    fs_insert_contains_eq(FiniteSet.empty[T], y, z)
                    (fs_insert(FiniteSet.empty[T], y).contains(z)
                        = (z = y or FiniteSet.empty[T].contains(z)))
                    finite_set_empty_contains_eq(z)
                    z = y
                    s.contains(z)
                }
                s.contains(z)
            }
            (fs_insert(fs_insert(FiniteSet.empty[T], y), x).contains(z) implies s.contains(z))
        }
        fs_subset_eq_intro(fs_insert(fs_insert(FiniteSet.empty[T], y), x), s)
        fs_insert(fs_insert(FiniteSet.empty[T], y), x).subset_eq(s)
        fs_card_mono(fs_insert(fs_insert(FiniteSet.empty[T], y), x), s)
        fs_card(fs_insert(fs_insert(FiniteSet.empty[T], y), x)) <= fs_card(s)
        Nat.2 <= fs_card(s)
    }
}

/// A finite set with positive cardinality has a member.
///
/// The counterpart of `fs_card_pos_of_contains`, which goes the other way. Read off a
/// duplicate-free list representation: a list of positive length is a cons, and its head is a
/// member.
theorem fs_member_of_card_pos[T](s: FiniteSet[T]) {
    Nat.1 <= fs_card(s) implies exists(x: T) { s.contains(x) }
} by {
    if Nat.1 <= fs_card(s) {
        fs_card_cardinality_is(s)
        s.cardinality_is(fs_card(s))
        finite_set_cardinality_has_exact_unique_list(s, fs_card(s))
        exists(items: List[T]) {
            fs_from_list(items) = s and items.is_unique and items.length = fs_card(s)
        }
        let (items: List[T]) satisfy {
            fs_from_list(items) = s and items.is_unique and items.length = fs_card(s)
        }
        match items {
            List.nil {
                List.nil[T].length = Nat.0
                Nat.1 <= Nat.0
                Nat.0 < Nat.1
                lte_and_lt(Nat.1, Nat.0, Nat.1)
                Nat.1 < Nat.1
                false
            }
            List.cons(head, tail) {
                List.cons(head, tail).contains(head)
                items.contains(head)
                fs_from_list_contains_eq(items, head)
                fs_from_list(items).contains(head)
                s.contains(head)
                exists(x: T) { s.contains(x) }
            }
        }
    }
}
