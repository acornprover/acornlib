from nat import Nat, divides_self, lt_and_lte
from number_theory import coprime_one_left, coprime_one_right, coprime_of_distinct_primes,
    central_binom
from data.nat.nat_range_prod import range_prod, range_prod_zero, range_prod_suc
from data.nat.nat_pairwise_coprime_divides import range_pairwise_coprime,
    range_pairwise_coprime_product_divides
from data.nat.nat_prime_central_binom import prime_in_interval_divides_central_binom

numerals Nat

/// A natural if it is prime, and one otherwise.
///
/// The device that makes a product over an interval of naturals stand in for a product over the
/// primes in it: composite positions contribute a factor of one, which is coprime to everything
/// and divides everything, so they neither disturb the coprimality nor the divisibility.
///
/// Written as a conditional here and nowhere else; the two case lemmas below are what everything
/// downstream uses.
define prime_or_one(k: Nat) -> Nat {
    if k.is_prime {
        k
    } else {
        Nat.1
    }
}

/// At a prime the factor is the prime.
theorem prime_or_one_prime(k: Nat) {
    k.is_prime implies prime_or_one(k) = k
}

/// Away from a prime the factor is one.
theorem prime_or_one_composite(k: Nat) {
    not k.is_prime implies prime_or_one(k) = Nat.1
}

/// Distinct positions give coprime factors.
///
/// Two distinct primes are coprime, and a factor of one is coprime to anything, so the only case
/// needing an argument is when both positions are prime.
theorem prime_or_one_coprime(j: Nat, k: Nat) {
    j != k implies prime_or_one(j).coprime(prime_or_one(k))
} by {
    if j != k {
        if j.is_prime {
            prime_or_one_prime(j)
            (prime_or_one(j) = j)
            if k.is_prime {
                prime_or_one_prime(k)
                (prime_or_one(k) = k)
                coprime_of_distinct_primes(j, k)
                j.coprime(k)
                prime_or_one(j).coprime(prime_or_one(k))
            }
            if not k.is_prime {
                prime_or_one_composite(k)
                (prime_or_one(k) = Nat.1)
                coprime_one_right(j)
                j.coprime(Nat.1)
                prime_or_one(j).coprime(prime_or_one(k))
            }
            prime_or_one(j).coprime(prime_or_one(k))
        }
        if not j.is_prime {
            prime_or_one_composite(j)
            (prime_or_one(j) = Nat.1)
            coprime_one_left(prime_or_one(k))
            Nat.1.coprime(prime_or_one(k))
            prime_or_one(j).coprime(prime_or_one(k))
        }
        prime_or_one(j).coprime(prime_or_one(k))
    }
}

/// The factor at the position `a + i` of an interval starting at `a`.
define interval_prime_fn(a: Nat, i: Nat) -> Nat {
    prime_or_one(a + i)
}

/// The factors over an interval are pairwise coprime.
///
/// Distinct offsets name distinct positions, so the factors are coprime.
theorem interval_prime_fn_pairwise_coprime(a: Nat, n: Nat) {
    range_pairwise_coprime(interval_prime_fn(a), n)
} by {
    forall(i: Nat, j: Nat) {
        if i < n and j < n and i != j {
            (a + i != a + j)
            prime_or_one_coprime(a + i, a + j)
            (prime_or_one(a + i).coprime(prime_or_one(a + j)))
            (interval_prime_fn(a, i) = prime_or_one(a + i))
            (interval_prime_fn(a, j) = prime_or_one(a + j))
            interval_prime_fn(a)(i).coprime(interval_prime_fn(a)(j))
        }
        (i < n and j < n and i != j
            implies interval_prime_fn(a)(i).coprime(interval_prime_fn(a)(j)))
    }
    (range_pairwise_coprime(interval_prime_fn(a), n) = forall(i: Nat, j: Nat) {
        i < n and j < n and i != j
            implies interval_prime_fn(a)(i).coprime(interval_prime_fn(a)(j))
    })
    range_pairwise_coprime(interval_prime_fn(a), n)
}

/// True if every prime in the interval `[a, a + n)` divides the given natural.
///
/// Named so that the product bound below is one quantifier deep.
define primes_in_interval_divide(a: Nat, n: Nat, x: Nat) -> Bool {
    forall(i: Nat) {
        i < n and (a + i).is_prime implies (a + i).divides(x)
    }
}

/// The product over an interval divides anything the primes in it all divide.
///
/// The product bound a Bertrand-style argument needs: each prime in the interval divides the
/// number, the primes are pairwise coprime because they are distinct, so their product divides
/// too. Composite positions contribute one and are carried along at no cost.
theorem interval_prime_product_divides(a: Nat, n: Nat, x: Nat) {
    primes_in_interval_divide(a, n, x)
        implies range_prod(interval_prime_fn(a), n).divides(x)
} by {
    if primes_in_interval_divide(a, n, x) {
        (primes_in_interval_divide(a, n, x) = forall(i: Nat) {
            i < n and (a + i).is_prime implies (a + i).divides(x)
        })
        interval_prime_fn_pairwise_coprime(a, n)
        range_pairwise_coprime(interval_prime_fn(a), n)
        forall(i: Nat) {
            if i < n {
                (interval_prime_fn(a, i) = prime_or_one(a + i))
                if (a + i).is_prime {
                    ((a + i).divides(x))
                    prime_or_one_prime(a + i)
                    (prime_or_one(a + i) = a + i)
                    interval_prime_fn(a)(i).divides(x)
                }
                if not (a + i).is_prime {
                    prime_or_one_composite(a + i)
                    (prime_or_one(a + i) = Nat.1)
                    Nat.1.divides(x)
                    interval_prime_fn(a)(i).divides(x)
                }
                interval_prime_fn(a)(i).divides(x)
            }
            (i < n implies interval_prime_fn(a)(i).divides(x))
        }
        range_pairwise_coprime_product_divides(interval_prime_fn(a), n, x)
        range_prod(interval_prime_fn(a), n).divides(x)
    }
}

/// The product of the primes above `n` and at most `2n` divides the central binomial coefficient.
///
/// The product bound at the interval Bertrand's argument uses. Each such prime divides the
/// central binomial coefficient, and distinct primes are coprime, so the whole product does. This
/// is the half of the argument that is about divisibility; what remains is the size estimate that
/// makes the product too large for the interval to be empty.
theorem central_binom_prime_interval_product_divides(m: Nat) {
    range_prod(interval_prime_fn(m.suc.suc), m.suc).divides(central_binom(m.suc))
} by {
    forall(i: Nat) {
        if i < m.suc and (m.suc.suc + i).is_prime {
            m.suc < m.suc.suc
            m.suc.suc <= m.suc.suc + i
            lt_and_lte(m.suc, m.suc.suc, m.suc.suc + i)
            m.suc < m.suc.suc + i
            i <= m
            (m.suc.suc + i <= m.suc.suc + m)
            (m.suc.suc + m = m.suc + m.suc)
            (m.suc.suc + i <= m.suc + m.suc)
            prime_in_interval_divides_central_binom(m.suc.suc + i, m)
            (m.suc.suc + i).divides(central_binom(m.suc))
        }
        (i < m.suc and (m.suc.suc + i).is_prime
            implies (m.suc.suc + i).divides(central_binom(m.suc)))
    }
    (primes_in_interval_divide(m.suc.suc, m.suc, central_binom(m.suc))
        = forall(i: Nat) {
        i < m.suc and (m.suc.suc + i).is_prime
            implies (m.suc.suc + i).divides(central_binom(m.suc))
    })
    primes_in_interval_divide(m.suc.suc, m.suc, central_binom(m.suc))
    interval_prime_product_divides(m.suc.suc, m.suc, central_binom(m.suc))
    range_prod(interval_prime_fn(m.suc.suc), m.suc).divides(central_binom(m.suc))
}
