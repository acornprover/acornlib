from nat import Nat, mul_assoc, mul_comm

numerals Nat

/// Swapping the inner two factors of a product of two products.
///
/// The rearrangement that turns a fact about each of two factorisations into a fact about
/// their product. It appears whenever two multiplicative structures are combined, and the
/// associativity and commutativity steps are tedious enough to be worth naming once.
theorem mul_swap_inner(a: Nat, b: Nat, c: Nat, d: Nat) {
    (a * b) * (c * d) = (a * c) * (b * d)
} by {
    mul_assoc(a, b, c * d)
    (a * b) * (c * d) = a * (b * (c * d))
    mul_assoc(b, c, d)
    b * (c * d) = (b * c) * d
    mul_comm(b, c)
    (b * c) * d = (c * b) * d
    mul_assoc(c, b, d)
    (c * b) * d = c * (b * d)
    b * (c * d) = c * (b * d)
    a * (b * (c * d)) = a * (c * (b * d))
    mul_assoc(a, c, b * d)
    a * (c * (b * d)) = (a * c) * (b * d)
}

/// Moving the outer left factor inside on the right.
theorem mul_shift_left(a: Nat, b: Nat, c: Nat) {
    (a * b) * c = (a * c) * b
} by {
    mul_assoc(a, b, c)
    (a * b) * c = a * (b * c)
    mul_comm(b, c)
    a * (b * c) = a * (c * b)
    mul_assoc(a, c, b)
    a * (c * b) = (a * c) * b
}

/// A product of a square with a square is the square of the product.
theorem mul_self_mul_self(a: Nat, b: Nat) {
    (a * a) * (b * b) = (a * b) * (a * b)
} by {
    mul_swap_inner(a, a, b, b)
}
