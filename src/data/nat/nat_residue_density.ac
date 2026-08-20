from nat import Nat, from_nat, div_mod_decomp, mod_lt
from real import Real, lt_mul_pos_right, lte_lt_trans, lte_trans, lte_antisymm,
    lte_mul_nonneg_left, mul_div_cancel, pos_gt_zero, lte_add_right, from_rat_pos
from data.nat.nat_counting_function import count_upto
from data.nat.nat_density import range_size, range_size_positive, density_seq
from analysis import from_nat_real_lte
from data.nat.nat_residue_count import in_residue_class
from data.nat.nat_residue_count_bound import count_upto_residue_bounds

/// A positive factor can be cancelled from an inequality.
///
/// `src/real/` has multiplication monotone in a nonnegative factor but not the converse, which
/// is what lets a cross-multiplied bound be read back as a bound on a quotient.
theorem lte_of_mul_lte_pos(x: Real, y: Real, c: Real) {
    x * c <= y * c and c > Real.0 implies x <= y
} by {
    if x * c <= y * c and c > Real.0 {
        if y < x {
            c.is_positive
            lt_mul_pos_right(y, x, c)
            y * c < x * c
            lte_lt_trans(x * c, y * c, x * c)
            x * c < x * c
            false
        }
        not (y < x)
        x <= y
    }
}

/// The density fraction times the range size is the count.
theorem density_seq_times_range(p: Nat -> Bool, n: Nat) {
    density_seq(p, n) * range_size(n) = from_nat[Real](count_upto(p, n.suc))
} by {
    range_size_positive(n)
    Real.0 < range_size(n)
    range_size(n) != Real.0
    mul_div_cancel(from_nat[Real](count_upto(p, n.suc)), range_size(n))
    (range_size(n) * (from_nat[Real](count_upto(p, n.suc)) / range_size(n))
        = from_nat[Real](count_upto(p, n.suc)))
    (density_seq(p, n) = from_nat[Real](count_upto(p, n.suc)) / range_size(n))
    (density_seq(p, n) * range_size(n)
        = range_size(n) * (from_nat[Real](count_upto(p, n.suc)) / range_size(n)))
    density_seq(p, n) * range_size(n) = from_nat[Real](count_upto(p, n.suc))
}

/// The modulus times the density fraction is within a fraction of one, from above.
///
/// The cross-multiplied form of the residue count bound: no division appears until the very
/// last step, where a positive factor is cancelled.
theorem residue_density_seq_upper(m: Nat, a: Nat, eps: Real, n: Nat) {
    a < m and eps > Real.0 and from_nat[Real](m) <= eps * range_size(n)
        implies from_nat[Real](m) * density_seq(in_residue_class(m, a), n)
            <= Real.1 + eps
} by {
    if a < m and eps > Real.0 and from_nat[Real](m) <= eps * range_size(n) {
        count_upto_residue_bounds(m, a, n.suc)
        (m * count_upto(in_residue_class(m, a), n.suc) <= n.suc + m)
        from_nat_real_lte(m * count_upto(in_residue_class(m, a), n.suc), n.suc + m)
        (from_nat[Real](m * count_upto(in_residue_class(m, a), n.suc))
            <= from_nat[Real](n.suc + m))
        (from_nat[Real](m * count_upto(in_residue_class(m, a), n.suc))
            = from_nat[Real](m) * from_nat[Real](count_upto(in_residue_class(m, a), n.suc)))
        (from_nat[Real](n.suc + m) = from_nat[Real](n.suc) + from_nat[Real](m))
        (range_size(n) = from_nat[Real](n.suc))
        (from_nat[Real](m) * from_nat[Real](count_upto(in_residue_class(m, a), n.suc))
            <= range_size(n) + from_nat[Real](m))
        (range_size(n) + from_nat[Real](m) <= range_size(n) + eps * range_size(n))
        (range_size(n) + eps * range_size(n) = (Real.1 + eps) * range_size(n))
        lte_trans(from_nat[Real](m)
            * from_nat[Real](count_upto(in_residue_class(m, a), n.suc)),
            range_size(n) + from_nat[Real](m), (Real.1 + eps) * range_size(n))
        (from_nat[Real](m) * from_nat[Real](count_upto(in_residue_class(m, a), n.suc))
            <= (Real.1 + eps) * range_size(n))
        density_seq_times_range(in_residue_class(m, a), n)
        (density_seq(in_residue_class(m, a), n) * range_size(n)
            = from_nat[Real](count_upto(in_residue_class(m, a), n.suc)))
        ((from_nat[Real](m) * density_seq(in_residue_class(m, a), n)) * range_size(n)
            = from_nat[Real](m)
                * (density_seq(in_residue_class(m, a), n) * range_size(n)))
        ((from_nat[Real](m) * density_seq(in_residue_class(m, a), n)) * range_size(n)
            <= (Real.1 + eps) * range_size(n))
        range_size_positive(n)
        range_size(n) > Real.0
        lte_of_mul_lte_pos(from_nat[Real](m) * density_seq(in_residue_class(m, a), n),
            Real.1 + eps, range_size(n))
        from_nat[Real](m) * density_seq(in_residue_class(m, a), n) <= Real.1 + eps
    }
}

/// The modulus times the density fraction is within a fraction of one, from below.
theorem residue_density_seq_lower(m: Nat, a: Nat, eps: Real, n: Nat) {
    a < m and eps > Real.0 and from_nat[Real](m) <= eps * range_size(n)
        implies Real.1 - eps
            <= from_nat[Real](m) * density_seq(in_residue_class(m, a), n)
} by {
    if a < m and eps > Real.0 and from_nat[Real](m) <= eps * range_size(n) {
        count_upto_residue_bounds(m, a, n.suc)
        (n.suc <= m * count_upto(in_residue_class(m, a), n.suc) + m)
        from_nat_real_lte(n.suc, m * count_upto(in_residue_class(m, a), n.suc) + m)
        (from_nat[Real](n.suc)
            <= from_nat[Real](m * count_upto(in_residue_class(m, a), n.suc) + m))
        (from_nat[Real](m * count_upto(in_residue_class(m, a), n.suc) + m)
            = from_nat[Real](m * count_upto(in_residue_class(m, a), n.suc))
                + from_nat[Real](m))
        (from_nat[Real](m * count_upto(in_residue_class(m, a), n.suc))
            = from_nat[Real](m) * from_nat[Real](count_upto(in_residue_class(m, a), n.suc)))
        (from_nat[Real](m * count_upto(in_residue_class(m, a), n.suc) + m)
            = from_nat[Real](m) * from_nat[Real](count_upto(in_residue_class(m, a), n.suc))
                + from_nat[Real](m))
        (range_size(n) = from_nat[Real](n.suc))
        (range_size(n)
            <= from_nat[Real](m) * from_nat[Real](count_upto(in_residue_class(m, a), n.suc))
                + from_nat[Real](m))
        density_seq_times_range(in_residue_class(m, a), n)
        (density_seq(in_residue_class(m, a), n) * range_size(n)
            = from_nat[Real](count_upto(in_residue_class(m, a), n.suc)))
        ((from_nat[Real](m) * density_seq(in_residue_class(m, a), n)) * range_size(n)
            = from_nat[Real](m)
                * (density_seq(in_residue_class(m, a), n) * range_size(n)))
        (range_size(n)
            <= (from_nat[Real](m) * density_seq(in_residue_class(m, a), n)) * range_size(n)
                + from_nat[Real](m))
        (from_nat[Real](m) <= eps * range_size(n))
        ((from_nat[Real](m) * density_seq(in_residue_class(m, a), n)) * range_size(n)
            + from_nat[Real](m)
            <= (from_nat[Real](m) * density_seq(in_residue_class(m, a), n)) * range_size(n)
                + eps * range_size(n))
        lte_trans(range_size(n),
            (from_nat[Real](m) * density_seq(in_residue_class(m, a), n)) * range_size(n)
                + from_nat[Real](m),
            (from_nat[Real](m) * density_seq(in_residue_class(m, a), n)) * range_size(n)
                + eps * range_size(n))
        (((from_nat[Real](m) * density_seq(in_residue_class(m, a), n)) * range_size(n)
            + eps * range_size(n))
            = ((from_nat[Real](m) * density_seq(in_residue_class(m, a), n)) + eps)
                * range_size(n))
        (Real.1 * range_size(n) = range_size(n))
        (range_size(n)
            <= ((from_nat[Real](m) * density_seq(in_residue_class(m, a), n)) + eps)
                * range_size(n))
        (Real.1 * range_size(n)
            <= ((from_nat[Real](m) * density_seq(in_residue_class(m, a), n)) + eps)
                * range_size(n))
        range_size_positive(n)
        range_size(n) > Real.0
        lte_of_mul_lte_pos(Real.1,
            (from_nat[Real](m) * density_seq(in_residue_class(m, a), n)) + eps,
            range_size(n))
        (Real.1 <= (from_nat[Real](m) * density_seq(in_residue_class(m, a), n)) + eps)
        lte_add_right(Real.1,
            (from_nat[Real](m) * density_seq(in_residue_class(m, a), n)) + eps, -eps)
        (Real.1 + -eps
            <= ((from_nat[Real](m) * density_seq(in_residue_class(m, a), n)) + eps) + -eps)
        (((from_nat[Real](m) * density_seq(in_residue_class(m, a), n)) + eps) + -eps
            = (from_nat[Real](m) * density_seq(in_residue_class(m, a), n))
                + (eps + -eps))
        (eps + -eps = Real.0)
        ((from_nat[Real](m) * density_seq(in_residue_class(m, a), n)) + Real.0
            = from_nat[Real](m) * density_seq(in_residue_class(m, a), n))
        (Real.1 + -eps = Real.1 - eps)
        (Real.1 - eps
            <= from_nat[Real](m) * density_seq(in_residue_class(m, a), n))
    }
}
