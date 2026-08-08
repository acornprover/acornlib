from nat import Nat, lt_and_lte, add_sub, add_imp_sub
from algebra.add_comm_monoid import AddCommMonoid
from list import partial
from data.nat.nat_range_sum_bridge import range_sum_eq_partial
from data.nat.nat_range_sum import range_sum, range_sum_zero, range_sum_suc, range_sum_congr,
    shift_fn, range_sum_split

numerals Nat

/// The summand read backwards over a range of length `n`.
///
/// Index `i` reads the value at `n - 1 - i`, written `n - i.suc` so that no separate subtraction
/// of one appears. Outside the range the value is whatever truncation gives, which nothing below
/// depends on: a range sum sees the summand only on its range.
define rev_fn[A: AddCommMonoid](f: Nat -> A, n: Nat, i: Nat) -> A {
    f(n - i.suc)
}

/// Reversing a range of length one more, past the first index, is reversing the shifted summand.
///
/// The step that makes the induction close: peeling the last index off the reversed sum leaves
/// the original's first value, and what is left below is the reversal of the summand shifted by
/// one.
theorem rev_fn_suc_below[A: AddCommMonoid](f: Nat -> A, n: Nat, i: Nat) {
    i < n implies rev_fn(f, n.suc, i) = rev_fn(shift_fn(f, Nat.1), n, i)
} by {
    if i < n {
        (rev_fn(f, n.suc, i) = f(n.suc - i.suc))
        i <= n
        add_sub(n, i)
        ((n - i) + i = n)
        ((n - i) + i.suc = n.suc)
        add_imp_sub(n - i, i.suc, n.suc)
        (n.suc - i.suc = n - i)
        i.suc <= n
        add_sub(n, i.suc)
        ((n - i.suc) + i.suc = n)
        (((n - i.suc) + Nat.1) + i = n)
        add_imp_sub((n - i.suc) + Nat.1, i, n)
        (n - i = (n - i.suc) + Nat.1)
        (rev_fn(shift_fn(f, Nat.1), n, i) = shift_fn(f, Nat.1)(n - i.suc))
        (shift_fn(f, Nat.1, n - i.suc) = f(Nat.1 + (n - i.suc)))
        (Nat.1 + (n - i.suc) = (n - i.suc) + Nat.1)
        rev_fn(f, n.suc, i) = rev_fn(shift_fn(f, Nat.1), n, i)
    }
}

/// The last index of a reversed range reads the first value.
theorem rev_fn_top[A: AddCommMonoid](f: Nat -> A, n: Nat) {
    rev_fn(f, n.suc, n) = f(Nat.0)
} by {
    (rev_fn(f, n.suc, n) = f(n.suc - n.suc))
    (n.suc - n.suc = Nat.0)
    rev_fn(f, n.suc, n) = f(Nat.0)
}

/// A range sum is unchanged by reading the summand backwards.
///
/// The reindexing identity a convolution needs: the sum over a range does not depend on the
/// direction it is taken in. Induction on the length, generalised over the summand, since the
/// step compares the reversal of a range with the reversal of the shifted summand one shorter.
theorem range_sum_rev[A: AddCommMonoid](f: Nat -> A, n: Nat) {
    range_sum(f, n) = range_sum(rev_fn(f, n), n)
} by {
    define w(k: Nat) -> Bool {
        forall(g: Nat -> A) {
            range_sum(g, k) = range_sum(rev_fn(g, k), k)
        }
    }
    forall(g: Nat -> A) {
        range_sum_zero(g)
        range_sum_zero(rev_fn(g, Nat.0))
        range_sum(g, Nat.0) = range_sum(rev_fn(g, Nat.0), Nat.0)
    }
    w(Nat.0)
    forall(k: Nat) {
        if w(k) {
            (forall(h: Nat -> A) {
                range_sum(h, k) = range_sum(rev_fn(h, k), k)
            })
            forall(g: Nat -> A) {
                (range_sum(shift_fn(g, Nat.1), k)
                    = range_sum(rev_fn(shift_fn(g, Nat.1), k), k))
                forall(i: Nat) {
                    if i < k {
                        rev_fn_suc_below(g, k, i)
                        (rev_fn(g, k.suc, i) = rev_fn(shift_fn(g, Nat.1), k, i))
                    }
                    (i < k implies rev_fn(g, k.suc)(i)
                        = rev_fn(shift_fn(g, Nat.1), k)(i))
                }
                range_sum_congr(rev_fn(g, k.suc), rev_fn(shift_fn(g, Nat.1), k), k)
                (range_sum(rev_fn(g, k.suc), k)
                    = range_sum(rev_fn(shift_fn(g, Nat.1), k), k))
                (range_sum(rev_fn(g, k.suc), k) = range_sum(shift_fn(g, Nat.1), k))
                range_sum_suc(rev_fn(g, k.suc), k)
                (range_sum(rev_fn(g, k.suc), k.suc)
                    = range_sum(rev_fn(g, k.suc), k) + rev_fn(g, k.suc)(k))
                rev_fn_top(g, k)
                (rev_fn(g, k.suc, k) = g(Nat.0))
                (range_sum(rev_fn(g, k.suc), k.suc)
                    = range_sum(shift_fn(g, Nat.1), k) + g(Nat.0))
                range_sum_split(g, Nat.1, k)
                (range_sum(g, Nat.1 + k)
                    = range_sum(g, Nat.1) + range_sum(shift_fn(g, Nat.1), k))
                (range_sum(g, Nat.1) = range_sum(g, Nat.0) + g(Nat.0))
                range_sum_zero(g)
                (range_sum(g, Nat.1) = g(Nat.0))
                (Nat.1 + k = k.suc)
                (range_sum(g, k.suc) = g(Nat.0) + range_sum(shift_fn(g, Nat.1), k))
                (g(Nat.0) + range_sum(shift_fn(g, Nat.1), k)
                    = range_sum(shift_fn(g, Nat.1), k) + g(Nat.0))
                range_sum(g, k.suc) = range_sum(rev_fn(g, k.suc), k.suc)
            }
            w(k.suc)
        }
        (w(k) implies w(k.suc))
    }
    w(Nat.0) and forall(k: Nat) {
        w(k) implies w(k.suc)
    }
    Nat.induction(w)
    w(n)
    (forall(g: Nat -> A) {
        range_sum(g, n) = range_sum(rev_fn(g, n), n)
    })
    range_sum(f, n) = range_sum(rev_fn(f, n), n)
}

/// A partial sum is unchanged by reading the summand backwards.
///
/// The same identity for the `partial` sums of `src/list/`, which is the form the polynomial
/// convolution is written in, so this is the shape a commutativity argument for it would cite.
theorem partial_rev[A: AddCommMonoid](f: Nat -> A, n: Nat) {
    partial(f, n) = partial(rev_fn(f, n), n)
} by {
    range_sum_eq_partial(f, n)
    (range_sum(f, n) = partial(f, n))
    range_sum_eq_partial(rev_fn(f, n), n)
    (range_sum(rev_fn(f, n), n) = partial(rev_fn(f, n), n))
    range_sum_rev(f, n)
    (range_sum(f, n) = range_sum(rev_fn(f, n), n))
    partial(f, n) = partial(rev_fn(f, n), n)
}
