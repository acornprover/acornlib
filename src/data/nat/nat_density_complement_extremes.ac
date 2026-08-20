from nat import Nat
from real import Real
from data.basic.functions import function_extensionality
from data.nat.nat_density import density_seq
from data.nat.nat_density_complement import not_pred, density_seq_complement
from analysis import is_bounded_above, limsup, is_bounded_below, liminf, one_minus_seq,
    limsup_one_minus
from data.nat.nat_density_extremes import upper_density, lower_density, density_seq_bounded_above,
    density_seq_bounded_below

/// The density sequence of a complement is the reflection of the density sequence.
theorem density_seq_complement_eq(p: Nat -> Bool) {
    density_seq(not_pred(p)) = one_minus_seq(density_seq(p))
} by {
    forall(n: Nat) {
        density_seq_complement(p, n)
        density_seq(not_pred(p), n) = Real.1 - density_seq(p, n)
        (density_seq(not_pred(p))(n) = density_seq(not_pred(p), n))
        (one_minus_seq(density_seq(p))(n) = Real.1 - density_seq(p)(n))
        (density_seq(p)(n) = density_seq(p, n))
        density_seq(not_pred(p))(n) = one_minus_seq(density_seq(p))(n)
    }
    function_extensionality[Nat, Real](density_seq(not_pred(p)),
        one_minus_seq(density_seq(p)))
    density_seq(not_pred(p)) = one_minus_seq(density_seq(p))
}

/// The upper density of a complement is one minus the lower density.
///
/// The counting identity makes the complement's density sequence the reflection of the
/// original's, and reflection exchanges the limit superior and the limit inferior.
theorem upper_density_complement(p: Nat -> Bool) {
    upper_density(not_pred(p)) = Real.1 - lower_density(p)
} by {
    density_seq_bounded_above(p)
    is_bounded_above(density_seq(p))
    density_seq_bounded_below(p)
    is_bounded_below(density_seq(p))
    limsup_one_minus(density_seq(p))
    limsup(one_minus_seq(density_seq(p))) = Real.1 - liminf(density_seq(p))
    density_seq_complement_eq(p)
    density_seq(not_pred(p)) = one_minus_seq(density_seq(p))
    limsup(density_seq(not_pred(p))) = Real.1 - liminf(density_seq(p))
    (upper_density(not_pred(p)) = limsup(density_seq(not_pred(p))))
    (lower_density(p) = liminf(density_seq(p)))
    upper_density(not_pred(p)) = Real.1 - lower_density(p)
}
