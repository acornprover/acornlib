from nat import Nat, div_mod_decomp, mod_lt, lte_and_lt, lt_and_lte, lte_add_right,
    lte_mul_both, lte_trans
from data.nat.nat_counting_function import count_upto
from data.nat.nat_residue_count import in_residue_class, count_upto_in_block_before,
    count_upto_in_block_after, count_upto_multiple

numerals Nat

/// The count of a residue class below a bound, in terms of the quotient.
///
/// Splitting the bound into whole blocks and a remainder, the count is the number of whole
/// blocks, plus one exactly when the remainder has passed the residue.
theorem count_upto_residue_cases(m: Nat, a: Nat, n: Nat) {
    a < m implies (count_upto(in_residue_class(m, a), n) = n.div(m)
        or count_upto(in_residue_class(m, a), n) = n.div(m) + Nat.1)
} by {
    if a < m {
        m != Nat.0
        mod_lt(n, m)
        n.mod(m) < m
        div_mod_decomp(n, m)
        n.div(m) * m + n.mod(m) = n
        count_upto_multiple(m, a, n.div(m))
        count_upto(in_residue_class(m, a), n.div(m) * m) = n.div(m)
        if n.mod(m) <= a {
            count_upto_in_block_before(m, a, n.div(m), n.mod(m))
            (count_upto(in_residue_class(m, a), n.div(m) * m + n.mod(m))
                = count_upto(in_residue_class(m, a), n.div(m) * m))
            count_upto(in_residue_class(m, a), n) = n.div(m)
        }
        if not (n.mod(m) <= a) {
            a < n.mod(m)
            n.mod(m) <= m
            count_upto_in_block_after(m, a, n.div(m), n.mod(m))
            (count_upto(in_residue_class(m, a), n.div(m) * m + n.mod(m))
                = count_upto(in_residue_class(m, a), n.div(m) * m) + Nat.1)
            count_upto(in_residue_class(m, a), n) = n.div(m) + Nat.1
        }
        (count_upto(in_residue_class(m, a), n) = n.div(m)
            or count_upto(in_residue_class(m, a), n) = n.div(m) + Nat.1)
    }
}

/// The count of a residue class is at least the number of whole blocks.
theorem count_upto_residue_lower(m: Nat, a: Nat, n: Nat) {
    a < m implies n.div(m) <= count_upto(in_residue_class(m, a), n)
} by {
    if a < m {
        count_upto_residue_cases(m, a, n)
        (count_upto(in_residue_class(m, a), n) = n.div(m)
            or count_upto(in_residue_class(m, a), n) = n.div(m) + Nat.1)
        if count_upto(in_residue_class(m, a), n) = n.div(m) {
            n.div(m) <= count_upto(in_residue_class(m, a), n)
        }
        if count_upto(in_residue_class(m, a), n) != n.div(m) {
            count_upto(in_residue_class(m, a), n) = n.div(m) + Nat.1
            n.div(m) <= n.div(m) + Nat.1
            n.div(m) <= count_upto(in_residue_class(m, a), n)
        }
        n.div(m) <= count_upto(in_residue_class(m, a), n)
    }
}

/// The count of a residue class is at most one more than the number of whole blocks.
theorem count_upto_residue_upper(m: Nat, a: Nat, n: Nat) {
    a < m implies count_upto(in_residue_class(m, a), n) <= n.div(m) + Nat.1
} by {
    if a < m {
        count_upto_residue_cases(m, a, n)
        (count_upto(in_residue_class(m, a), n) = n.div(m)
            or count_upto(in_residue_class(m, a), n) = n.div(m) + Nat.1)
        if count_upto(in_residue_class(m, a), n) = n.div(m) {
            n.div(m) <= n.div(m) + Nat.1
            count_upto(in_residue_class(m, a), n) <= n.div(m) + Nat.1
        }
        if count_upto(in_residue_class(m, a), n) != n.div(m) {
            count_upto(in_residue_class(m, a), n) = n.div(m) + Nat.1
            count_upto(in_residue_class(m, a), n) <= n.div(m) + Nat.1
        }
        count_upto(in_residue_class(m, a), n) <= n.div(m) + Nat.1
    }
}

/// The count of a residue class, scaled by the modulus, does not exceed the bound by much.
///
/// The multiplicative form of the sandwich, which avoids division on the real side: the count
/// times the modulus is within one modulus of the bound in each direction.
theorem count_upto_residue_bounds(m: Nat, a: Nat, n: Nat) {
    a < m implies m * count_upto(in_residue_class(m, a), n) <= n + m
        and n <= m * count_upto(in_residue_class(m, a), n) + m
} by {
    if a < m {
        m != Nat.0
        div_mod_decomp(n, m)
        n.div(m) * m + n.mod(m) = n
        mod_lt(n, m)
        n.mod(m) < m
        count_upto_residue_upper(m, a, n)
        count_upto(in_residue_class(m, a), n) <= n.div(m) + Nat.1
        lte_mul_both(m, count_upto(in_residue_class(m, a), n), n.div(m) + Nat.1)
        m * count_upto(in_residue_class(m, a), n) <= m * (n.div(m) + Nat.1)
        (m * (n.div(m) + Nat.1) = n.div(m) * m + m)
        (n.div(m) * m <= n)
        (n.div(m) * m + m <= n + m)
        lte_trans(m * count_upto(in_residue_class(m, a), n), n.div(m) * m + m, n + m)
        m * count_upto(in_residue_class(m, a), n) <= n + m
        count_upto_residue_lower(m, a, n)
        n.div(m) <= count_upto(in_residue_class(m, a), n)
        lte_mul_both(m, n.div(m), count_upto(in_residue_class(m, a), n))
        m * n.div(m) <= m * count_upto(in_residue_class(m, a), n)
        (m * n.div(m) = n.div(m) * m)
        (n.mod(m) <= m)
        (n.div(m) * m + n.mod(m) <= n.div(m) * m + m)
        (n <= n.div(m) * m + m)
        (n.div(m) * m + m <= m * count_upto(in_residue_class(m, a), n) + m)
        lte_trans(n, n.div(m) * m + m, m * count_upto(in_residue_class(m, a), n) + m)
        n <= m * count_upto(in_residue_class(m, a), n) + m
        (m * count_upto(in_residue_class(m, a), n) <= n + m
            and n <= m * count_upto(in_residue_class(m, a), n) + m)
    }
}
