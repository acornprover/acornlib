from nat import Nat, has_prime_divisor, divides_trans, divides_self, lt_and_lte
from data.nat.nat_square import is_square, is_square_intro
from data.nat.nat_squarefree import is_squarefree, is_squarefree_apply, is_squarefree_intro

numerals Nat

/// True if no square of a prime divides the natural.
///
/// The prime form of squarefreeness. Named so that the equivalence below is a statement between
/// two one-quantifier conditions.
define no_prime_square_divides(n: Nat) -> Bool {
    forall(p: Nat) {
        p.is_prime implies not (p * p).divides(n)
    }
}

/// A prime whose square divides refutes the condition.
theorem no_prime_square_divides_apply(n: Nat, p: Nat) {
    no_prime_square_divides(n) and p.is_prime implies not (p * p).divides(n)
} by {
    if no_prime_square_divides(n) and p.is_prime {
        (no_prime_square_divides(n) = forall(q: Nat) {
            q.is_prime implies not (q * q).divides(n)
        })
        (p.is_prime implies not (p * p).divides(n))
        not (p * p).divides(n)
    }
}

/// The pointwise condition gives the prime form.
theorem no_prime_square_divides_intro(n: Nat) {
    (forall(p: Nat) { p.is_prime implies not (p * p).divides(n) })
        implies no_prime_square_divides(n)
} by {
    if forall(p: Nat) { p.is_prime implies not (p * p).divides(n) } {
        (no_prime_square_divides(n) = forall(q: Nat) {
            q.is_prime implies not (q * q).divides(n)
        })
        no_prime_square_divides(n)
    }
}

/// A squarefree natural has no prime square dividing it.
///
/// The square of a prime is a square other than one, so it cannot divide.
theorem no_prime_square_divides_of_squarefree(n: Nat) {
    is_squarefree(n) implies no_prime_square_divides(n)
} by {
    if is_squarefree(n) {
        forall(p: Nat) {
            if p.is_prime {
                Nat.1 < p
                if (p * p).divides(n) {
                    is_square_intro(p * p, p)
                    is_square(p * p)
                    is_squarefree_apply(n, p * p)
                    p * p = Nat.1
                    Nat.1 <= p
                    (Nat.1 * Nat.1 = Nat.1)
                    (Nat.1 <= p * p)
                    (p <= p * p)
                    lt_and_lte(Nat.1, p, p * p)
                    Nat.1 < p * p
                    not (Nat.1 < Nat.1)
                    false
                }
                not (p * p).divides(n)
            }
            (p.is_prime implies not (p * p).divides(n))
        }
        no_prime_square_divides_intro(n)
        no_prime_square_divides(n)
    }
}

/// A natural with no prime square dividing it is squarefree.
///
/// A square divisor other than one has a prime factor, and the square of that prime divides the
/// square divisor, hence the number. So the prime form is no weaker, and squarefreeness needs no
/// prime valuations to be characterised through primes.
theorem squarefree_of_no_prime_square_divides(n: Nat) {
    no_prime_square_divides(n) implies is_squarefree(n)
} by {
    if no_prime_square_divides(n) {
        forall(d: Nat) {
            if is_square(d) and d.divides(n) {
                (is_square(d) = exists(r: Nat) { r * r = d })
                let (r: Nat) satisfy {
                    r * r = d
                }
                if d != Nat.1 {
                    if r = Nat.0 {
                        (r * r = Nat.0)
                        d = Nat.0
                    }
                    if r = Nat.1 {
                        (r * r = Nat.1)
                        d = Nat.1
                        false
                    }
                    r != Nat.1
                    if r = Nat.0 {
                        d = Nat.0
                        (Nat.0.divides(n) = exists(z: Nat) { Nat.0 * z = n })
                        let (z: Nat) satisfy {
                            Nat.0 * z = n
                        }
                        (Nat.0 * z = Nat.0)
                        n = Nat.0
                        (Nat.1.suc = Nat.2)
                        Nat.1 < Nat.2
                        has_prime_divisor(Nat.2)
                        exists(q: Nat) { q.is_prime and q.divides(Nat.2) }
                        let (q: Nat) satisfy {
                            q.is_prime and q.divides(Nat.2)
                        }
                        ((q * q) * Nat.0 = Nat.0)
                        exists(c: Nat) { (q * q) * c = n }
                        ((q * q).divides(n) = exists(c: Nat) { (q * q) * c = n })
                        (q * q).divides(n)
                        no_prime_square_divides_apply(n, q)
                        not (q * q).divides(n)
                        false
                    }
                    r != Nat.0
                    Nat.1 < r
                    has_prime_divisor(r)
                    exists(p: Nat) { p.is_prime and p.divides(r) }
                    let (p: Nat) satisfy {
                        p.is_prime and p.divides(r)
                    }
                    (p.divides(r) = exists(c: Nat) { p * c = r })
                    let (e: Nat) satisfy {
                        p * e = r
                    }
                    ((p * p) * (e * e) = (p * e) * (p * e))
                    ((p * e) * (p * e) = r * r)
                    ((p * p) * (e * e) = d)
                    exists(c: Nat) { (p * p) * c = d }
                    ((p * p).divides(d) = exists(c: Nat) { (p * p) * c = d })
                    (p * p).divides(d)
                    divides_trans(p * p, d, n)
                    (p * p).divides(n)
                    no_prime_square_divides_apply(n, p)
                    not (p * p).divides(n)
                    false
                }
                d = Nat.1
            }
            (is_square(d) and d.divides(n) implies d = Nat.1)
        }
        is_squarefree_intro(n)
        is_squarefree(n)
    }
}

/// Squarefreeness is exactly the absence of a prime square divisor.
///
/// The characterisation the prime valuation form is usually stated as, reached without any
/// factorisation machinery: only that a natural above one has a prime divisor.
theorem is_squarefree_iff_no_prime_square_divides(n: Nat) {
    is_squarefree(n) = no_prime_square_divides(n)
} by {
    if is_squarefree(n) {
        no_prime_square_divides_of_squarefree(n)
        no_prime_square_divides(n)
    }
    if no_prime_square_divides(n) {
        squarefree_of_no_prime_square_divides(n)
        is_squarefree(n)
    }
    is_squarefree(n) = no_prime_square_divides(n)
}
