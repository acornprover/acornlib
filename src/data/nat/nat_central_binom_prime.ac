from nat import Nat, divides_lte, lt_or_lte, lte_antisymm, lt_and_lte, lte_and_lt,
    lt_trans, lte_trans, only_zero_lte_zero, add_to_zero
from number_theory import central_binom, central_binom_ne_zero, central_binom_legendre,
    prime_factor_count_upto, prime_factor_count_upto_zero, prime_factor_count_upto_step,
    count_prime_factor, prime_divides_iff_count_ne_zero

numerals Nat

/// A prime above a positive natural does not divide it.
///
/// A divisor of a positive natural is at most that natural.
theorem prime_above_not_divides(p: Nat, m: Nat) {
    m != Nat.0 and m < p implies not p.divides(m)
} by {
    if m != Nat.0 and m < p {
        if p.divides(m) {
            divides_lte(p, m)
            (m = Nat.0 or p <= m)
            p <= m
            lte_and_lt(p, m, p)
            p < p
            false
        }
        not p.divides(m)
    }
}

/// A prime above a positive natural has valuation zero there.
theorem count_prime_factor_zero_of_lt(p: Nat, m: Nat) {
    p.is_prime and m != Nat.0 and m < p implies count_prime_factor(p, m) = Nat.0
} by {
    if p.is_prime and m != Nat.0 and m < p {
        prime_above_not_divides(p, m)
        not p.divides(m)
        prime_divides_iff_count_ne_zero(p, m)
        p.divides(m) = (count_prime_factor(p, m) != Nat.0)
        not (count_prime_factor(p, m) != Nat.0)
        count_prime_factor(p, m) = Nat.0
    }
}

/// A prime above the whole range contributes nothing to the running valuation total.
///
/// Every term of the total is the valuation at some positive natural below the prime, and
/// each of those is zero.
theorem prime_factor_count_upto_zero_of_lt(p: Nat, n: Nat) {
    p.is_prime and n < p implies prime_factor_count_upto(p, n) = Nat.0
} by {
    define q(x: Nat) -> Bool {
        x < p implies prime_factor_count_upto(p, x) = Nat.0
    }
    prime_factor_count_upto_zero(p)
    prime_factor_count_upto(p, Nat.0) = Nat.0
    q(Nat.0)
    forall(k: Nat) {
        if q(k) {
            if k.suc < p {
                k < k.suc
                lt_trans(k, k.suc, p)
                k < p
                prime_factor_count_upto(p, k) = Nat.0
                k.suc != Nat.0
                count_prime_factor_zero_of_lt(p, k.suc)
                count_prime_factor(p, k.suc) = Nat.0
                prime_factor_count_upto_step(p, k)
                prime_factor_count_upto(p, k.suc) =
                    prime_factor_count_upto(p, k) + count_prime_factor(p, k.suc)
                prime_factor_count_upto(p, k.suc) = Nat.0 + Nat.0
                Nat.0 + Nat.0 = Nat.0
                prime_factor_count_upto(p, k.suc) = Nat.0
            }
            q(k.suc)
        }
        (q(k) implies q(k.suc))
    }
    q(Nat.0) and forall(k: Nat) {
        q(k) implies q(k.suc)
    }
    Nat.induction(q)
    q(n)
}

/// A prime at most the top of the range contributes to the running valuation total.
///
/// The prime is one of the naturals counted, and it has positive valuation at itself, so the
/// total cannot be zero.
theorem prime_factor_count_upto_ne_zero(p: Nat, n: Nat) {
    p.is_prime and p <= n implies prime_factor_count_upto(p, n) != Nat.0
} by {
    define q(x: Nat) -> Bool {
        p <= x implies prime_factor_count_upto(p, x) != Nat.0
    }
    if p <= Nat.0 {
        only_zero_lte_zero(p)
        p = Nat.0
        p.is_prime = (Nat.1 < p and not p.is_composite)
        Nat.1 < p
        Nat.1 < Nat.0
        Nat.0 <= Nat.1
        lte_and_lt(Nat.0, Nat.1, Nat.0)
        Nat.0 < Nat.0
        false
    }
    not (p <= Nat.0)
    q(Nat.0)
    forall(k: Nat) {
        if q(k) {
            if p <= k.suc {
                prime_factor_count_upto_step(p, k)
                prime_factor_count_upto(p, k.suc) =
                    prime_factor_count_upto(p, k) + count_prime_factor(p, k.suc)
                if p <= k {
                    prime_factor_count_upto(p, k) != Nat.0
                    if prime_factor_count_upto(p, k.suc) = Nat.0 {
                        (prime_factor_count_upto(p, k) + count_prime_factor(p, k.suc)
                            = Nat.0)
                        add_to_zero(prime_factor_count_upto(p, k),
                            count_prime_factor(p, k.suc))
                        prime_factor_count_upto(p, k) = Nat.0
                        false
                    }
                    prime_factor_count_upto(p, k.suc) != Nat.0
                }
                if not (p <= k) {
                    lt_or_lte(k, p)
                    k < p
                    lte_antisymm(p, k.suc)
                    p = k.suc
                    p.divides(p)
                    p != Nat.0
                    prime_divides_iff_count_ne_zero(p, p)
                    p.divides(p) = (count_prime_factor(p, p) != Nat.0)
                    count_prime_factor(p, p) != Nat.0
                    count_prime_factor(p, k.suc) != Nat.0
                    if prime_factor_count_upto(p, k.suc) = Nat.0 {
                        (prime_factor_count_upto(p, k) + count_prime_factor(p, k.suc)
                            = Nat.0)
                        add_to_zero(prime_factor_count_upto(p, k),
                            count_prime_factor(p, k.suc))
                        count_prime_factor(p, k.suc) = Nat.0
                        false
                    }
                    prime_factor_count_upto(p, k.suc) != Nat.0
                }
                prime_factor_count_upto(p, k.suc) != Nat.0
            }
            q(k.suc)
        }
        (q(k) implies q(k.suc))
    }
    q(Nat.0) and forall(k: Nat) {
        q(k) implies q(k.suc)
    }
    Nat.induction(q)
    q(n)
}

/// A prime in the upper half of the range divides the central binomial coefficient.
///
/// Legendre's identity puts the valuation at the coefficient together with two copies of the
/// running total up to `n`. A prime above `n` contributes nothing to those, so the whole of
/// the total up to `2n` sits at the coefficient, and that total is nonzero because the prime
/// is one of the naturals it counts.
theorem prime_in_interval_divides_central_binom(n: Nat, p: Nat) {
    p.is_prime and n < p and p <= n + n implies p.divides(central_binom(n))
} by {
    if p.is_prime and n < p and p <= n + n {
        prime_factor_count_upto_zero_of_lt(p, n)
        prime_factor_count_upto(p, n) = Nat.0
        central_binom_legendre(p, n)
        (count_prime_factor(p, central_binom(n)) + prime_factor_count_upto(p, n)
            + prime_factor_count_upto(p, n) = prime_factor_count_upto(p, n + n))
        (count_prime_factor(p, central_binom(n)) + Nat.0 + Nat.0 =
            prime_factor_count_upto(p, n + n))
        count_prime_factor(p, central_binom(n)) = prime_factor_count_upto(p, n + n)
        prime_factor_count_upto_ne_zero(p, n + n)
        prime_factor_count_upto(p, n + n) != Nat.0
        count_prime_factor(p, central_binom(n)) != Nat.0
        central_binom_ne_zero(n)
        central_binom(n) != Nat.0
        prime_divides_iff_count_ne_zero(p, central_binom(n))
        p.divides(central_binom(n)) = (count_prime_factor(p, central_binom(n)) != Nat.0)
        p.divides(central_binom(n))
    }
}

/// Every prime in the upper half of the range divides the central binomial coefficient.
///
/// The classical statement, which is the input to the standard bound on the product of the
/// primes in an interval.
theorem primes_in_interval_divide_central_binom(n: Nat) {
    forall(p: Nat) {
        (p.is_prime and n < p and p <= n + n implies p.divides(central_binom(n)))
    }
} by {
    forall(p: Nat) {
        if p.is_prime and n < p and p <= n + n {
            prime_in_interval_divides_central_binom(n, p)
            p.divides(central_binom(n))
        }
        (p.is_prime and n < p and p <= n + n implies p.divides(central_binom(n)))
    }
}
