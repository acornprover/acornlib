from nat import Nat, from_nat
from real import Real, add_seq, lte_trans, mul_div_cancel, mul_left_cancel
from data.nat.nat_counting_function import count_upto, count_upto_zero, count_upto_suc_true,
    count_upto_suc_false
from data.nat.nat_density import range_size, range_size_positive, density_seq
from analysis import is_bounded_above, is_bounded_below, liminf, add_seq_bounded_above,
    add_seq_bounded_below, liminf_add
from data.nat.nat_density_mono import liminf_mono
from data.nat.nat_density_extremes import lower_density, density_seq_bounded_above,
    density_seq_bounded_below
from data.nat.nat_density_subadditive import or_pred, or_pred_apply

numerals Nat

/// True if no natural satisfies both predicates.
define disjoint_pred(p: Nat -> Bool, r: Nat -> Bool) -> Bool {
    forall(i: Nat) {
        not (p(i) and r(i))
    }
}

/// Disjointness applies at each index.
theorem disjoint_pred_apply(p: Nat -> Bool, r: Nat -> Bool, i: Nat) {
    disjoint_pred(p, r) implies not (p(i) and r(i))
} by {
    if disjoint_pred(p, r) {
        (disjoint_pred(p, r) = forall(j: Nat) { not (p(j) and r(j)) })
        forall(j: Nat) {
            not (p(j) and r(j))
        }
        not (p(i) and r(i))
    }
}

/// A disjoint union is counted exactly as often as the two parts together.
///
/// The inequality of `count_upto_or_le` becomes an equality once no index is counted twice.
theorem count_upto_or_disjoint(p: Nat -> Bool, r: Nat -> Bool, n: Nat) {
    disjoint_pred(p, r)
        implies count_upto(or_pred(p, r), n) = count_upto(p, n) + count_upto(r, n)
} by {
    if disjoint_pred(p, r) {
        define q(x: Nat) -> Bool {
            count_upto(or_pred(p, r), x) = count_upto(p, x) + count_upto(r, x)
        }
        count_upto_zero(or_pred(p, r))
        count_upto_zero(p)
        count_upto_zero(r)
        (Nat.0 = Nat.0 + Nat.0)
        q(Nat.0)
        forall(k: Nat) {
            if q(k) {
                count_upto(or_pred(p, r), k) = count_upto(p, k) + count_upto(r, k)
                or_pred_apply(p, r, k)
                (or_pred(p, r)(k) = (p(k) or r(k)))
                disjoint_pred_apply(p, r, k)
                not (p(k) and r(k))
                if p(k) {
                    not r(k)
                    or_pred(p, r)(k)
                    count_upto_suc_true(or_pred(p, r), k)
                    (count_upto(or_pred(p, r), k.suc)
                        = count_upto(or_pred(p, r), k) + Nat.1)
                    count_upto_suc_true(p, k)
                    count_upto(p, k.suc) = count_upto(p, k) + Nat.1
                    count_upto_suc_false(r, k)
                    count_upto(r, k.suc) = count_upto(r, k)
                    ((count_upto(p, k) + count_upto(r, k)) + Nat.1
                        = (count_upto(p, k) + Nat.1) + count_upto(r, k))
                    (count_upto(or_pred(p, r), k.suc)
                        = count_upto(p, k.suc) + count_upto(r, k.suc))
                }
                if not p(k) {
                    count_upto_suc_false(p, k)
                    count_upto(p, k.suc) = count_upto(p, k)
                    if r(k) {
                        or_pred(p, r)(k)
                        count_upto_suc_true(or_pred(p, r), k)
                        (count_upto(or_pred(p, r), k.suc)
                            = count_upto(or_pred(p, r), k) + Nat.1)
                        count_upto_suc_true(r, k)
                        count_upto(r, k.suc) = count_upto(r, k) + Nat.1
                        ((count_upto(p, k) + count_upto(r, k)) + Nat.1
                            = count_upto(p, k) + (count_upto(r, k) + Nat.1))
                        (count_upto(or_pred(p, r), k.suc)
                            = count_upto(p, k.suc) + count_upto(r, k.suc))
                    }
                    if not r(k) {
                        not or_pred(p, r)(k)
                        count_upto_suc_false(or_pred(p, r), k)
                        (count_upto(or_pred(p, r), k.suc) = count_upto(or_pred(p, r), k))
                        count_upto_suc_false(r, k)
                        count_upto(r, k.suc) = count_upto(r, k)
                        (count_upto(or_pred(p, r), k.suc)
                            = count_upto(p, k.suc) + count_upto(r, k.suc))
                    }
                    (count_upto(or_pred(p, r), k.suc)
                        = count_upto(p, k.suc) + count_upto(r, k.suc))
                }
                (count_upto(or_pred(p, r), k.suc)
                    = count_upto(p, k.suc) + count_upto(r, k.suc))
                q(k.suc)
            }
            (q(k) implies q(k.suc))
        }
        q(Nat.0) and forall(k: Nat) {
            q(k) implies q(k.suc)
        }
        Nat.induction(q)
        q(n)
        count_upto(or_pred(p, r), n) = count_upto(p, n) + count_upto(r, n)
    }
}

/// The density fraction of a disjoint union is the sum of the two fractions.
theorem density_seq_or_disjoint(p: Nat -> Bool, r: Nat -> Bool, n: Nat) {
    disjoint_pred(p, r)
        implies density_seq(or_pred(p, r), n) = density_seq(p, n) + density_seq(r, n)
} by {
    if disjoint_pred(p, r) {
        count_upto_or_disjoint(p, r, n.suc)
        (count_upto(or_pred(p, r), n.suc) = count_upto(p, n.suc) + count_upto(r, n.suc))
        (from_nat[Real](count_upto(or_pred(p, r), n.suc))
            = from_nat[Real](count_upto(p, n.suc) + count_upto(r, n.suc)))
        (from_nat[Real](count_upto(p, n.suc) + count_upto(r, n.suc))
            = from_nat[Real](count_upto(p, n.suc)) + from_nat[Real](count_upto(r, n.suc)))
        (from_nat[Real](count_upto(or_pred(p, r), n.suc))
            = from_nat[Real](count_upto(p, n.suc)) + from_nat[Real](count_upto(r, n.suc)))
        range_size_positive(n)
        Real.0 < range_size(n)
        range_size(n) != Real.0
        mul_div_cancel(from_nat[Real](count_upto(p, n.suc)), range_size(n))
        mul_div_cancel(from_nat[Real](count_upto(r, n.suc)), range_size(n))
        mul_div_cancel(from_nat[Real](count_upto(or_pred(p, r), n.suc)), range_size(n))
        (range_size(n) * (from_nat[Real](count_upto(p, n.suc)) / range_size(n)
            + from_nat[Real](count_upto(r, n.suc)) / range_size(n))
            = range_size(n) * (from_nat[Real](count_upto(p, n.suc)) / range_size(n))
                + range_size(n) * (from_nat[Real](count_upto(r, n.suc)) / range_size(n)))
        (range_size(n) * (from_nat[Real](count_upto(p, n.suc)) / range_size(n)
            + from_nat[Real](count_upto(r, n.suc)) / range_size(n))
            = from_nat[Real](count_upto(or_pred(p, r), n.suc)))
        mul_left_cancel(from_nat[Real](count_upto(or_pred(p, r), n.suc)), range_size(n),
            from_nat[Real](count_upto(p, n.suc)) / range_size(n)
                + from_nat[Real](count_upto(r, n.suc)) / range_size(n))
        (from_nat[Real](count_upto(or_pred(p, r), n.suc)) / range_size(n)
            = from_nat[Real](count_upto(p, n.suc)) / range_size(n)
                + from_nat[Real](count_upto(r, n.suc)) / range_size(n))
        (density_seq(or_pred(p, r), n)
            = from_nat[Real](count_upto(or_pred(p, r), n.suc)) / range_size(n))
        (density_seq(p, n) = from_nat[Real](count_upto(p, n.suc)) / range_size(n))
        (density_seq(r, n) = from_nat[Real](count_upto(r, n.suc)) / range_size(n))
        density_seq(or_pred(p, r), n) = density_seq(p, n) + density_seq(r, n)
    }
}

/// The lower density is superadditive over a disjoint union.
///
/// With the upper density bounded by one, this is what bounds the weights of a disjoint system.
theorem lower_density_superadditive(p: Nat -> Bool, r: Nat -> Bool) {
    disjoint_pred(p, r)
        implies lower_density(p) + lower_density(r) <= lower_density(or_pred(p, r))
} by {
    if disjoint_pred(p, r) {
        density_seq_bounded_above(p)
        is_bounded_above(density_seq(p))
        density_seq_bounded_below(p)
        is_bounded_below(density_seq(p))
        density_seq_bounded_above(r)
        is_bounded_above(density_seq(r))
        density_seq_bounded_below(r)
        is_bounded_below(density_seq(r))
        density_seq_bounded_above(or_pred(p, r))
        is_bounded_above(density_seq(or_pred(p, r)))
        density_seq_bounded_below(or_pred(p, r))
        is_bounded_below(density_seq(or_pred(p, r)))
        add_seq_bounded_above(density_seq(p), density_seq(r))
        is_bounded_above(add_seq(density_seq(p), density_seq(r)))
        add_seq_bounded_below(density_seq(p), density_seq(r))
        is_bounded_below(add_seq(density_seq(p), density_seq(r)))
        forall(n: Nat) {
            density_seq_or_disjoint(p, r, n)
            density_seq(or_pred(p, r), n) = density_seq(p, n) + density_seq(r, n)
            (density_seq(or_pred(p, r))(n) = density_seq(or_pred(p, r), n))
            (density_seq(p)(n) = density_seq(p, n))
            (density_seq(r)(n) = density_seq(r, n))
            (add_seq(density_seq(p), density_seq(r))(n)
                = density_seq(p)(n) + density_seq(r)(n))
            (add_seq(density_seq(p), density_seq(r))(n)
                <= density_seq(or_pred(p, r))(n))
        }
        liminf_mono(add_seq(density_seq(p), density_seq(r)), density_seq(or_pred(p, r)))
        (liminf(add_seq(density_seq(p), density_seq(r)))
            <= liminf(density_seq(or_pred(p, r))))
        liminf_add(density_seq(p), density_seq(r))
        (liminf(density_seq(p)) + liminf(density_seq(r))
            <= liminf(add_seq(density_seq(p), density_seq(r))))
        lte_trans(liminf(density_seq(p)) + liminf(density_seq(r)),
            liminf(add_seq(density_seq(p), density_seq(r))),
            liminf(density_seq(or_pred(p, r))))
        (liminf(density_seq(p)) + liminf(density_seq(r))
            <= liminf(density_seq(or_pred(p, r))))
        (lower_density(p) = liminf(density_seq(p)))
        (lower_density(r) = liminf(density_seq(r)))
        (lower_density(or_pred(p, r)) = liminf(density_seq(or_pred(p, r))))
        lower_density(p) + lower_density(r) <= lower_density(or_pred(p, r))
    }
}
