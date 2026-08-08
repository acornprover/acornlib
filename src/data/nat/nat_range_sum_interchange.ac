from nat import Nat
from algebra.add_comm_monoid import AddCommMonoid
from semiring import Semiring
from list import partial
from data.nat.nat_range_sum_bridge import range_sum_eq_partial
from algebra.add_semigroup import add_fn
from data.basic.functions import function_extensionality
from data.nat.nat_range_sum import range_sum, range_sum_zero, range_sum_suc, range_sum_congr,
    range_sum_add, zero_fn, range_sum_zero_fn

numerals Nat

/// The sum along one row of a doubly indexed family.
///
/// The family is curried rather than taken on pairs, so that a row is a function of one variable
/// and the inner sum is an ordinary range sum.
define row_sum[A: AddCommMonoid](g: Nat -> (Nat -> A), n: Nat, i: Nat) -> A {
    range_sum(g(i), n)
}

/// One column of a doubly indexed family, as a function of the row.
define col_fn[A: AddCommMonoid](g: Nat -> (Nat -> A), j: Nat, i: Nat) -> A {
    g(i)(j)
}

/// The sum along one column of a doubly indexed family.
define col_sum[A: AddCommMonoid](g: Nat -> (Nat -> A), m: Nat, j: Nat) -> A {
    range_sum(col_fn(g, j), m)
}

/// Extending the rows by one adds that row to every column sum.
theorem col_sum_suc[A: AddCommMonoid](g: Nat -> (Nat -> A), m: Nat) {
    col_sum(g, m.suc) = add_fn(col_sum(g, m), g(m))
} by {
    forall(j: Nat) {
        (col_sum(g, m.suc)(j) = range_sum(col_fn(g, j), m.suc))
        range_sum_suc(col_fn(g, j), m)
        (range_sum(col_fn(g, j), m.suc) = range_sum(col_fn(g, j), m) + col_fn(g, j)(m))
        (col_fn(g, j, m) = g(m)(j))
        (col_sum(g, m)(j) = range_sum(col_fn(g, j), m))
        (add_fn(col_sum(g, m), g(m))(j) = col_sum(g, m)(j) + g(m)(j))
        (col_sum(g, m.suc)(j) = add_fn(col_sum(g, m), g(m))(j))
    }
    function_extensionality[Nat, A](col_sum(g, m.suc), add_fn(col_sum(g, m), g(m)))
    col_sum(g, m.suc) = add_fn(col_sum(g, m), g(m))
}

/// A double range sum may be taken in either order.
///
/// Fubini for range sums: summing the rows and summing the columns give the same answer.
/// Induction on the number of rows, where one more row adds itself to every column sum at once
/// and the sum of a pointwise sum splits.
theorem range_sum_interchange[A: AddCommMonoid](g: Nat -> (Nat -> A), m: Nat, n: Nat) {
    range_sum(row_sum(g, n), m) = range_sum(col_sum(g, m), n)
} by {
    define p(k: Nat) -> Bool {
        range_sum(row_sum(g, n), k) = range_sum(col_sum(g, k), n)
    }
    range_sum_zero(row_sum(g, n))
    (range_sum(row_sum(g, n), Nat.0) = A.0)
    forall(j: Nat) {
        (col_sum(g, Nat.0)(j) = range_sum(col_fn(g, j), Nat.0))
        range_sum_zero(col_fn(g, j))
        (range_sum(col_fn(g, j), Nat.0) = A.0)
        (zero_fn[A](j) = A.0)
        (j < n implies col_sum(g, Nat.0)(j) = zero_fn[A](j))
    }
    range_sum_congr(col_sum(g, Nat.0), zero_fn[A], n)
    (range_sum(col_sum(g, Nat.0), n) = range_sum(zero_fn[A], n))
    range_sum_zero_fn[A](n)
    (range_sum(zero_fn[A], n) = A.0)
    p(Nat.0)
    forall(k: Nat) {
        if p(k) {
            (range_sum(row_sum(g, n), k) = range_sum(col_sum(g, k), n))
            range_sum_suc(row_sum(g, n), k)
            (range_sum(row_sum(g, n), k.suc)
                = range_sum(row_sum(g, n), k) + row_sum(g, n)(k))
            (row_sum(g, n, k) = range_sum(g(k), n))
            col_sum_suc(g, k)
            (col_sum(g, k.suc) = add_fn(col_sum(g, k), g(k)))
            range_sum_add(col_sum(g, k), g(k), n)
            (range_sum(add_fn(col_sum(g, k), g(k)), n)
                = range_sum(col_sum(g, k), n) + range_sum(g(k), n))
            (range_sum(col_sum(g, k.suc), n)
                = range_sum(col_sum(g, k), n) + range_sum(g(k), n))
            range_sum(row_sum(g, n), k.suc) = range_sum(col_sum(g, k.suc), n)
            p(k.suc)
        }
        (p(k) implies p(k.suc))
    }
    p(Nat.0) and forall(k: Nat) {
        p(k) implies p(k.suc)
    }
    Nat.induction(p)
    p(m)
}

/// A double partial sum may be taken in either order.
///
/// The same identity for the `partial` sums of `src/list/`, which is the form the polynomial
/// convolution is written in.
theorem partial_interchange[A: AddCommMonoid](g: Nat -> (Nat -> A), m: Nat, n: Nat) {
    partial(row_sum(g, n), m) = partial(col_sum(g, m), n)
} by {
    range_sum_eq_partial(row_sum(g, n), m)
    (range_sum(row_sum(g, n), m) = partial(row_sum(g, n), m))
    range_sum_eq_partial(col_sum(g, m), n)
    (range_sum(col_sum(g, m), n) = partial(col_sum(g, m), n))
    range_sum_interchange(g, m, n)
    (range_sum(row_sum(g, n), m) = range_sum(col_sum(g, m), n))
    partial(row_sum(g, n), m) = partial(col_sum(g, m), n)
}

/// A summand scaled on the left by a fixed factor.
define scale_fn[R: Semiring](c: R, f: Nat -> R, i: Nat) -> R {
    c * f(i)
}

/// A constant factor comes out of a range sum.
///
/// Distributivity applied along the range. This is the other half of what a convolution
/// rearrangement needs: the interchange moves the sums past each other, and this moves a factor
/// of the outer sum inside the inner one.
theorem range_sum_scale[R: Semiring](c: R, f: Nat -> R, n: Nat) {
    range_sum(scale_fn(c, f), n) = c * range_sum(f, n)
} by {
    define p(k: Nat) -> Bool {
        range_sum(scale_fn(c, f), k) = c * range_sum(f, k)
    }
    range_sum_zero(scale_fn(c, f))
    range_sum_zero(f)
    (c * R.0 = R.0)
    p(Nat.0)
    forall(k: Nat) {
        if p(k) {
            (range_sum(scale_fn(c, f), k) = c * range_sum(f, k))
            range_sum_suc(scale_fn(c, f), k)
            (range_sum(scale_fn(c, f), k.suc)
                = range_sum(scale_fn(c, f), k) + scale_fn(c, f)(k))
            (scale_fn(c, f, k) = c * f(k))
            range_sum_suc(f, k)
            (range_sum(f, k.suc) = range_sum(f, k) + f(k))
            (c * (range_sum(f, k) + f(k)) = c * range_sum(f, k) + c * f(k))
            range_sum(scale_fn(c, f), k.suc) = c * range_sum(f, k.suc)
            p(k.suc)
        }
        (p(k) implies p(k.suc))
    }
    p(Nat.0) and forall(k: Nat) {
        p(k) implies p(k.suc)
    }
    Nat.induction(p)
    p(n)
}
