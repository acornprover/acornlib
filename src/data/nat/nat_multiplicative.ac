from nat import Nat, gcd_one_left, gcd_one_right, mul_comm
from data.nat.nat_mul_rearrange import mul_swap_inner

numerals Nat

/// True if `f` sends one to one and turns products of coprime arguments into products.
///
/// The standard notion of a multiplicative arithmetic function. The value at a coprime
/// product is determined by the values at the factors, which is what lets such a function
/// be reconstructed from its values at prime powers.
define is_multiplicative(f: Nat -> Nat) -> Bool {
    f(Nat.1) = Nat.1 and forall(m: Nat, n: Nat) {
        m.coprime(n) implies f(m * n) = f(m) * f(n)
    }
}

/// A multiplicative function splits over a coprime product.
theorem is_multiplicative_apply(f: Nat -> Nat, m: Nat, n: Nat) {
    is_multiplicative(f) and m.coprime(n) implies f(m * n) = f(m) * f(n)
} by {
    if is_multiplicative(f) and m.coprime(n) {
        is_multiplicative(f) = (f(Nat.1) = Nat.1 and forall(a: Nat, b: Nat) {
            a.coprime(b) implies f(a * b) = f(a) * f(b)
        })
        forall(a: Nat, b: Nat) {
            a.coprime(b) implies f(a * b) = f(a) * f(b)
        }
        m.coprime(n) implies f(m * n) = f(m) * f(n)
        f(m * n) = f(m) * f(n)
    }
}

/// A multiplicative function takes one to one.
theorem is_multiplicative_one(f: Nat -> Nat) {
    is_multiplicative(f) implies f(Nat.1) = Nat.1
} by {
    if is_multiplicative(f) {
        is_multiplicative(f) = (f(Nat.1) = Nat.1 and forall(a: Nat, b: Nat) {
            a.coprime(b) implies f(a * b) = f(a) * f(b)
        })
        f(Nat.1) = Nat.1
    }
}

/// The two defining conditions give multiplicativity.
theorem is_multiplicative_intro(f: Nat -> Nat) {
    f(Nat.1) = Nat.1 and
    (forall(m: Nat, n: Nat) { m.coprime(n) implies f(m * n) = f(m) * f(n) })
        implies is_multiplicative(f)
} by {
    if f(Nat.1) = Nat.1 and
        forall(m: Nat, n: Nat) { m.coprime(n) implies f(m * n) = f(m) * f(n) } {
        is_multiplicative(f) = (f(Nat.1) = Nat.1 and forall(a: Nat, b: Nat) {
            a.coprime(b) implies f(a * b) = f(a) * f(b)
        })
        is_multiplicative(f)
    }
}

/// True if `f` sends one to one and turns every product into a product.
///
/// Stronger than multiplicativity, since no coprimality is required.
define is_completely_multiplicative(f: Nat -> Nat) -> Bool {
    f(Nat.1) = Nat.1 and forall(m: Nat, n: Nat) {
        f(m * n) = f(m) * f(n)
    }
}

/// A completely multiplicative function splits over every product.
theorem is_completely_multiplicative_apply(f: Nat -> Nat, m: Nat, n: Nat) {
    is_completely_multiplicative(f) implies f(m * n) = f(m) * f(n)
} by {
    if is_completely_multiplicative(f) {
        is_completely_multiplicative(f) = (f(Nat.1) = Nat.1 and forall(a: Nat, b: Nat) {
            f(a * b) = f(a) * f(b)
        })
        forall(a: Nat, b: Nat) {
            f(a * b) = f(a) * f(b)
        }
        f(m * n) = f(m) * f(n)
    }
}

/// A completely multiplicative function takes one to one.
theorem is_completely_multiplicative_one(f: Nat -> Nat) {
    is_completely_multiplicative(f) implies f(Nat.1) = Nat.1
} by {
    if is_completely_multiplicative(f) {
        is_completely_multiplicative(f) = (f(Nat.1) = Nat.1 and forall(a: Nat, b: Nat) {
            f(a * b) = f(a) * f(b)
        })
        f(Nat.1) = Nat.1
    }
}

/// The two defining conditions give complete multiplicativity.
theorem is_completely_multiplicative_intro(f: Nat -> Nat) {
    f(Nat.1) = Nat.1 and (forall(m: Nat, n: Nat) { f(m * n) = f(m) * f(n) })
        implies is_completely_multiplicative(f)
} by {
    if f(Nat.1) = Nat.1 and forall(m: Nat, n: Nat) { f(m * n) = f(m) * f(n) } {
        is_completely_multiplicative(f) = (f(Nat.1) = Nat.1 and forall(a: Nat, b: Nat) {
            f(a * b) = f(a) * f(b)
        })
        is_completely_multiplicative(f)
    }
}

/// Complete multiplicativity is stronger than multiplicativity.
theorem multiplicative_of_completely_multiplicative(f: Nat -> Nat) {
    is_completely_multiplicative(f) implies is_multiplicative(f)
} by {
    if is_completely_multiplicative(f) {
        is_completely_multiplicative_one(f)
        f(Nat.1) = Nat.1
        forall(m: Nat, n: Nat) {
            if m.coprime(n) {
                is_completely_multiplicative_apply(f, m, n)
                f(m * n) = f(m) * f(n)
            }
            m.coprime(n) implies f(m * n) = f(m) * f(n)
        }
        is_multiplicative_intro(f)
        is_multiplicative(f)
    }
}

/// The identity on the naturals.
define id_fn(n: Nat) -> Nat {
    n
}

/// The identity is completely multiplicative.
theorem id_fn_completely_multiplicative {
    is_completely_multiplicative(id_fn)
} by {
    id_fn(Nat.1) = Nat.1
    forall(m: Nat, n: Nat) {
        id_fn(m * n) = m * n
        id_fn(m) * id_fn(n) = m * n
        id_fn(m * n) = id_fn(m) * id_fn(n)
    }
    is_completely_multiplicative_intro(id_fn)
    is_completely_multiplicative(id_fn)
}

/// The function that is one everywhere.
define one_fn(n: Nat) -> Nat {
    Nat.1
}

/// The constant one is completely multiplicative.
theorem one_fn_completely_multiplicative {
    is_completely_multiplicative(one_fn)
} by {
    one_fn(Nat.1) = Nat.1
    forall(m: Nat, n: Nat) {
        one_fn(m * n) = Nat.1
        one_fn(m) * one_fn(n) = Nat.1 * Nat.1
        Nat.1 * Nat.1 = Nat.1
        one_fn(m * n) = one_fn(m) * one_fn(n)
    }
    is_completely_multiplicative_intro(one_fn)
    is_completely_multiplicative(one_fn)
}

/// The pointwise product of two arithmetic functions.
define mul_fn(f: Nat -> Nat, g: Nat -> Nat, n: Nat) -> Nat {
    f(n) * g(n)
}

/// The pointwise product of multiplicative functions is multiplicative.
///
/// The four values regroup by swapping the inner two factors.
theorem mul_fn_multiplicative(f: Nat -> Nat, g: Nat -> Nat) {
    is_multiplicative(f) and is_multiplicative(g) implies is_multiplicative(mul_fn(f, g))
} by {
    if is_multiplicative(f) and is_multiplicative(g) {
        is_multiplicative_one(f)
        f(Nat.1) = Nat.1
        is_multiplicative_one(g)
        g(Nat.1) = Nat.1
        mul_fn(f, g, Nat.1) = f(Nat.1) * g(Nat.1)
        mul_fn(f, g)(Nat.1) = Nat.1
        forall(m: Nat, n: Nat) {
            if m.coprime(n) {
                is_multiplicative_apply(f, m, n)
                f(m * n) = f(m) * f(n)
                is_multiplicative_apply(g, m, n)
                g(m * n) = g(m) * g(n)
                mul_fn(f, g, m * n) = f(m * n) * g(m * n)
                mul_fn(f, g)(m * n) = (f(m) * f(n)) * (g(m) * g(n))
                mul_swap_inner(f(m), f(n), g(m), g(n))
                (f(m) * f(n)) * (g(m) * g(n)) = (f(m) * g(m)) * (f(n) * g(n))
                mul_fn(f, g, m) = f(m) * g(m)
                mul_fn(f, g, n) = f(n) * g(n)
                mul_fn(f, g)(m * n) = mul_fn(f, g)(m) * mul_fn(f, g)(n)
            }
            m.coprime(n) implies mul_fn(f, g)(m * n) = mul_fn(f, g)(m) * mul_fn(f, g)(n)
        }
        is_multiplicative_intro(mul_fn(f, g))
        is_multiplicative(mul_fn(f, g))
    }
}

/// The pointwise product of completely multiplicative functions is completely multiplicative.
theorem mul_fn_completely_multiplicative(f: Nat -> Nat, g: Nat -> Nat) {
    is_completely_multiplicative(f) and is_completely_multiplicative(g)
        implies is_completely_multiplicative(mul_fn(f, g))
} by {
    if is_completely_multiplicative(f) and is_completely_multiplicative(g) {
        is_completely_multiplicative_one(f)
        f(Nat.1) = Nat.1
        is_completely_multiplicative_one(g)
        g(Nat.1) = Nat.1
        mul_fn(f, g, Nat.1) = f(Nat.1) * g(Nat.1)
        mul_fn(f, g)(Nat.1) = Nat.1
        forall(m: Nat, n: Nat) {
            is_completely_multiplicative_apply(f, m, n)
            f(m * n) = f(m) * f(n)
            is_completely_multiplicative_apply(g, m, n)
            g(m * n) = g(m) * g(n)
            mul_fn(f, g, m * n) = f(m * n) * g(m * n)
            mul_fn(f, g)(m * n) = (f(m) * f(n)) * (g(m) * g(n))
            mul_swap_inner(f(m), f(n), g(m), g(n))
            (f(m) * f(n)) * (g(m) * g(n)) = (f(m) * g(m)) * (f(n) * g(n))
            mul_fn(f, g, m) = f(m) * g(m)
            mul_fn(f, g, n) = f(n) * g(n)
            mul_fn(f, g)(m * n) = mul_fn(f, g)(m) * mul_fn(f, g)(n)
        }
        is_completely_multiplicative_intro(mul_fn(f, g))
        is_completely_multiplicative(mul_fn(f, g))
    }
}

/// A multiplicative function is unchanged by multiplying its argument by one.
///
/// One is coprime to everything, so this is the degenerate case of the splitting.
theorem multiplicative_mul_one(f: Nat -> Nat, n: Nat) {
    is_multiplicative(f) implies f(Nat.1 * n) = f(n)
} by {
    if is_multiplicative(f) {
        gcd_one_left(n)
        Nat.1.gcd(n) = Nat.1
        Nat.1.coprime(n) = (Nat.1.gcd(n) = Nat.1)
        Nat.1.coprime(n)
        is_multiplicative_apply(f, Nat.1, n)
        f(Nat.1 * n) = f(Nat.1) * f(n)
        is_multiplicative_one(f)
        f(Nat.1) = Nat.1
        f(Nat.1 * n) = Nat.1 * f(n)
        Nat.1 * f(n) = f(n)
    }
}
