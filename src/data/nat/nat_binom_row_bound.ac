from nat import Nat, pow_one, pow_zero, one_pow, lte_trans, lte_and_lt
from data.basic.functions import function_extensionality
from list import partial
from combinatorics import binom, binomial, binomial_term
from data.nat.nat_range_sum import range_sum, range_sum_zero, range_sum_suc
from data.nat.nat_range_sum_bridge import range_sum_eq_partial

numerals Nat

/// A single term is at most the whole sum, over the naturals.
///
/// Nothing states this, though it is what every bound of one term by a total needs. It holds
/// because the naturals have no negative summands to cancel the term against.
theorem range_sum_term_lte(f: Nat -> Nat, n: Nat, k: Nat) {
    k < n implies f(k) <= range_sum(f, n)
} by {
    define p(x: Nat) -> Bool {
        k < x implies f(k) <= range_sum(f, x)
    }
    not (k < Nat.0)
    p(Nat.0)
    forall(j: Nat) {
        if p(j) {
            if k < j.suc {
                range_sum_suc(f, j)
                (range_sum(f, j.suc) = range_sum(f, j) + f(j))
                k <= j
                if k = j {
                    (f(k) <= range_sum(f, j) + f(k))
                    f(k) <= range_sum(f, j.suc)
                }
                if k != j {
                    k < j
                    (k < j implies f(k) <= range_sum(f, j))
                    f(k) <= range_sum(f, j)
                    (range_sum(f, j) <= range_sum(f, j) + f(j))
                    lte_trans(f(k), range_sum(f, j), range_sum(f, j.suc))
                    f(k) <= range_sum(f, j.suc)
                }
                f(k) <= range_sum(f, j.suc)
            }
            p(j.suc)
        }
        (p(j) implies p(j.suc))
    }
    p(Nat.0) and forall(j: Nat) {
        p(j) implies p(j.suc)
    }
    Nat.induction(p)
    p(n)
}

/// A single term is at most the whole partial sum.
theorem partial_term_lte(f: Nat -> Nat, n: Nat, k: Nat) {
    k < n implies f(k) <= partial(f, n)
} by {
    if k < n {
        range_sum_term_lte(f, n, k)
        (f(k) <= range_sum(f, n))
        range_sum_eq_partial(f, n)
        (range_sum(f, n) = partial(f, n))
        f(k) <= partial(f, n)
    }
}

/// The binomial coefficients of one row, as a function of the lower index.
define binom_row(m: Nat, k: Nat) -> Nat {
    m.binom(k)
}

/// At one and one the binomial term is the coefficient itself.
theorem binomial_term_one_one(m: Nat, k: Nat) {
    binomial_term(Nat.1, Nat.1, m, k) = binom_row(m, k)
} by {
    (binomial_term(Nat.1, Nat.1, m, k)
        = m.binom(k) * Nat.1.pow(k) * Nat.1.pow(m - k))
    one_pow[Nat](k)
    (Nat.1.pow(k) = Nat.1)
    one_pow[Nat](m - k)
    (Nat.1.pow(m - k) = Nat.1)
    (m.binom(k) * Nat.1 * Nat.1 = m.binom(k))
    (binom_row(m, k) = m.binom(k))
    binomial_term(Nat.1, Nat.1, m, k) = binom_row(m, k)
}

/// A row of binomial coefficients sums to a power of two.
///
/// The binomial theorem at one and one, where every power collapses.
theorem binom_row_sum(m: Nat) {
    partial(binom_row(m), m.suc) = Nat.2.pow(m)
} by {
    forall(k: Nat) {
        binomial_term_one_one(m, k)
        (binomial_term(Nat.1, Nat.1, m)(k) = binom_row(m)(k))
    }
    function_extensionality[Nat, Nat](binomial_term(Nat.1, Nat.1, m), binom_row(m))
    (binomial_term(Nat.1, Nat.1, m) = binom_row(m))
    binomial(Nat.1, Nat.1, m)
    ((Nat.1 + Nat.1).pow(m) = partial(binomial_term(Nat.1, Nat.1, m), m.suc))
    (Nat.1 + Nat.1 = Nat.2)
    partial(binom_row(m), m.suc) = Nat.2.pow(m)
}

/// Every binomial coefficient is at most two to the row index.
///
/// The standard bound: a coefficient is one term of a row that sums to a power of two, and over
/// the naturals a term never exceeds its sum.
theorem binom_lte_two_pow(m: Nat, k: Nat) {
    k <= m implies m.binom(k) <= Nat.2.pow(m)
} by {
    if k <= m {
        m < m.suc
        lte_and_lt(k, m, m.suc)
        k < m.suc
        partial_term_lte(binom_row(m), m.suc, k)
        (binom_row(m)(k) <= partial(binom_row(m), m.suc))
        (binom_row(m, k) = m.binom(k))
        binom_row_sum(m)
        (partial(binom_row(m), m.suc) = Nat.2.pow(m))
        m.binom(k) <= Nat.2.pow(m)
    }
}
