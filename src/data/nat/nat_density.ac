from nat import Nat, from_nat, lte_trans
from real import Real, converges_to, converges_to_unique, div_le_of_mul_le,
    mul_div_cancel
from analysis import from_nat_real_nonnegative, from_nat_real_lte, div_le_div_pos
from data.nat.nat_counting_function import count_upto, count_upto_le, count_upto_all,
    count_upto_none, count_upto_mono_pred, complement_pred, all_pred, none_pred

numerals Real

/// The size of the range used at index `n`.
///
/// One more than the index, so that the range is never empty and the ratio below never
/// divides by zero. The limit is unaffected by the shift.
define range_size(n: Nat) -> Real {
    from_nat[Real](n.suc)
}

/// The range size is positive.
theorem range_size_positive(n: Nat) {
    Real.0 < range_size(n)
} by {
    from_nat[Real](n.suc) > Real.0
    Real.0 < range_size(n)
}

/// The fraction of the first `n + 1` naturals satisfying `p`.
///
/// The sequence whose limit, when it exists, is the natural density.
define density_seq(p: Nat -> Bool, n: Nat) -> Real {
    from_nat[Real](count_upto(p, n.suc)) / range_size(n)
}

/// True if the fraction of naturals satisfying `p` tends to `d`.
///
/// The natural density in the usual sense. It need not exist; when it does, it is unique.
define has_natural_density(p: Nat -> Bool, d: Real) -> Bool {
    converges_to(density_seq(p), d)
}

/// The density sequence of such a predicate converges to the density.
theorem has_natural_density_apply(p: Nat -> Bool, d: Real) {
    has_natural_density(p, d) implies converges_to(density_seq(p), d)
} by {
    if has_natural_density(p, d) {
        has_natural_density(p, d) = converges_to(density_seq(p), d)
        converges_to(density_seq(p), d)
    }
}

/// Convergence of the density sequence is the density.
theorem has_natural_density_intro(p: Nat -> Bool, d: Real) {
    converges_to(density_seq(p), d) implies has_natural_density(p, d)
} by {
    if converges_to(density_seq(p), d) {
        has_natural_density(p, d) = converges_to(density_seq(p), d)
        has_natural_density(p, d)
    }
}

/// A predicate has at most one natural density.
///
/// The density is a limit, and limits of a real sequence are unique.
theorem natural_density_unique(p: Nat -> Bool, d: Real, e: Real) {
    has_natural_density(p, d) and has_natural_density(p, e) implies d = e
} by {
    if has_natural_density(p, d) and has_natural_density(p, e) {
        has_natural_density_apply(p, d)
        converges_to(density_seq(p), d)
        has_natural_density_apply(p, e)
        converges_to(density_seq(p), e)
        converges_to_unique(density_seq(p), d, e)
        d = e
    }
}

/// The everywhere-true predicate has density sequence constantly one.
theorem density_seq_all(n: Nat) {
    density_seq(all_pred, n) = Real.1
} by {
    count_upto_all(n.suc)
    count_upto(all_pred, n.suc) = n.suc
    density_seq(all_pred, n) = from_nat[Real](n.suc) / range_size(n)
    range_size(n) = from_nat[Real](n.suc)
    range_size_positive(n)
    Real.0 < range_size(n)
    range_size(n) != Real.0
    range_size(n) / range_size(n) = Real.1
    density_seq(all_pred, n) = Real.1
}

/// The nowhere-true predicate has density sequence constantly zero.
theorem density_seq_none(n: Nat) {
    density_seq(none_pred, n) = Real.0
} by {
    count_upto_none(n.suc)
    count_upto(none_pred, n.suc) = Nat.0
    from_nat[Real](Nat.0) = Real.0
    density_seq(none_pred, n) = Real.0 / range_size(n)
    Real.0 / range_size(n) = Real.0
    density_seq(none_pred, n) = Real.0
}

/// The density sequence never exceeds one.
///
/// At most every natural in the range satisfies the predicate.
theorem density_seq_le_one(p: Nat -> Bool, n: Nat) {
    density_seq(p, n) <= Real.1
} by {
    count_upto_le(p, n.suc)
    count_upto(p, n.suc) <= n.suc
    from_nat[Real](count_upto(p, n.suc)) <= from_nat[Real](n.suc)
    range_size(n) = from_nat[Real](n.suc)
    from_nat[Real](count_upto(p, n.suc)) <= range_size(n)
    range_size_positive(n)
    Real.0 < range_size(n)
    from_nat[Real](count_upto(p, n.suc)) / range_size(n) <= range_size(n) / range_size(n)
    range_size(n) != Real.0
    range_size(n) / range_size(n) = Real.1
    density_seq(p, n) <= Real.1
}

/// The density sequence is never negative.
theorem density_seq_nonnegative(p: Nat -> Bool, n: Nat) {
    Real.0 <= density_seq(p, n)
} by {
    from_nat_real_nonnegative(count_upto(p, n.suc))
    Real.0 <= from_nat[Real](count_upto(p, n.suc))
    range_size_positive(n)
    Real.0 < range_size(n)
    div_le_div_pos(Real.0, from_nat[Real](count_upto(p, n.suc)), range_size(n))
    Real.0 / range_size(n) <= from_nat[Real](count_upto(p, n.suc)) / range_size(n)
    Real.0 / range_size(n) = Real.0
    Real.0 <= from_nat[Real](count_upto(p, n.suc)) / range_size(n)
    Real.0 <= density_seq(p, n)
}

/// A predicate implied by another has a smaller density sequence.
///
/// Stated with the implication holding everywhere, which is the form the comparison is used
/// in; the counting lemma underneath needs it only below the range.
theorem density_seq_mono(p: Nat -> Bool, r: Nat -> Bool, n: Nat) {
    (forall(i: Nat) { p(i) implies r(i) }) implies density_seq(p, n) <= density_seq(r, n)
} by {
    if forall(i: Nat) { p(i) implies r(i) } {
        forall(i: Nat) {
            (p(i) implies r(i))
            (i < n.suc implies (p(i) implies r(i)))
        }
        count_upto_mono_pred(p, r, n.suc)
        count_upto(p, n.suc) <= count_upto(r, n.suc)
        from_nat[Real](count_upto(p, n.suc)) <= from_nat[Real](count_upto(r, n.suc))
        range_size_positive(n)
        Real.0 < range_size(n)
        (from_nat[Real](count_upto(p, n.suc)) / range_size(n)
            <= from_nat[Real](count_upto(r, n.suc)) / range_size(n))
        density_seq(p, n) <= density_seq(r, n)
    }
}

/// True if `p` holds from some point on.
///
/// The other standard reading of "almost all" on the naturals. It is strictly stronger than
/// having density one: a predicate can fail infinitely often and still have density one, so
/// long as its failures thin out.
define eventually_all(p: Nat -> Bool) -> Bool {
    exists(n: Nat) {
        forall(m: Nat) {
            n <= m implies p(m)
        }
    }
}

/// A point beyond which the predicate holds witnesses the condition.
theorem eventually_all_intro(p: Nat -> Bool, n: Nat) {
    (forall(m: Nat) { n <= m implies p(m) }) implies eventually_all(p)
} by {
    if forall(m: Nat) { n <= m implies p(m) } {
        exists(k: Nat) {
            forall(m: Nat) {
                k <= m implies p(m)
            }
        }
        eventually_all(p) = exists(k: Nat) {
            forall(m: Nat) {
                k <= m implies p(m)
            }
        }
        eventually_all(p)
    }
}

/// Such a point can be extracted.
theorem eventually_all_witness(p: Nat -> Bool) {
    eventually_all(p) implies exists(n: Nat) {
        forall(m: Nat) {
            n <= m implies p(m)
        }
    }
} by {
    if eventually_all(p) {
        eventually_all(p) = exists(k: Nat) {
            forall(m: Nat) {
                k <= m implies p(m)
            }
        }
        exists(k: Nat) {
            forall(m: Nat) {
                k <= m implies p(m)
            }
        }
    }
}

/// A predicate holding everywhere holds eventually.
theorem eventually_all_of_all(p: Nat -> Bool) {
    (forall(m: Nat) { p(m) }) implies eventually_all(p)
} by {
    if forall(m: Nat) { p(m) } {
        forall(m: Nat) {
            p(m)
            (Nat.0 <= m implies p(m))
        }
        eventually_all_intro(p, Nat.0)
        eventually_all(p)
    }
}

/// Eventual truth passes to a weaker predicate.
theorem eventually_all_mono(p: Nat -> Bool, r: Nat -> Bool) {
    eventually_all(p) and (forall(i: Nat) { p(i) implies r(i) }) implies eventually_all(r)
} by {
    if eventually_all(p) and forall(i: Nat) { p(i) implies r(i) } {
        eventually_all_witness(p)
        let (n: Nat) satisfy {
            forall(m: Nat) {
                n <= m implies p(m)
            }
        }
        forall(m: Nat) {
            if n <= m {
                p(m)
                r(m)
            }
            (n <= m implies r(m))
        }
        eventually_all_intro(r, n)
        eventually_all(r)
    }
}

/// The pointwise conjunction of two predicates on the naturals.
define and_pred(p: Nat -> Bool, r: Nat -> Bool, i: Nat) -> Bool {
    p(i) and r(i)
}

/// Two eventually-true predicates are eventually true together.
///
/// Beyond the later of the two thresholds both hold.
theorem eventually_all_and(p: Nat -> Bool, r: Nat -> Bool) {
    eventually_all(p) and eventually_all(r)
        implies eventually_all(and_pred(p, r))
} by {
    if eventually_all(p) and eventually_all(r) {
        eventually_all_witness(p)
        let (n: Nat) satisfy {
            forall(m: Nat) {
                n <= m implies p(m)
            }
        }
        eventually_all_witness(r)
        let (k: Nat) satisfy {
            forall(m: Nat) {
                k <= m implies r(m)
            }
        }
        forall(m: Nat) {
            if n.max(k) <= m {
                n <= n.max(k)
                lte_trans(n, n.max(k), m)
                n <= m
                p(m)
                k <= n.max(k)
                lte_trans(k, n.max(k), m)
                k <= m
                r(m)
                and_pred(p, r, m) = (p(m) and r(m))
                and_pred(p, r)(m)
            }
            (n.max(k) <= m implies and_pred(p, r)(m))
        }
        eventually_all_intro(and_pred(p, r), n.max(k))
        eventually_all(and_pred(p, r))
    }
}
