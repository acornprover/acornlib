from nat import Nat
from real import Real, lte_trans, lte_antisymm, converges, limit, seq_lte,
    seq_lte_preserves_limit
from data.nat.nat_density import density_seq, density_seq_mono
from analysis import tail_sup, tail_sup_bounds_terms, tail_sup_is_least,
    is_bounded_above, limsup, limsup_le_tail_sup, is_bounded_below, neg_tail_sup,
    neg_tail_sup_converges, neg_reverses_lte, liminf, tail_inf, tail_inf_bounds_terms,
    tail_inf_is_greatest, liminf_ge_tail_inf, neg_seq, neg_seq_bounded_above,
    neg_seq_bounded_below, liminf_eq_limit_tail_inf, neg_tail_sup_neg_seq_eq_tail_inf,
    lte_of_lte_add_eps
from data.nat.nat_density_extremes import upper_density, lower_density, density_seq_bounded_above,
    density_seq_bounded_below

/// A tail supremum is monotone in the sequence.
///
/// Proved at the level of the tails, where it is one application of each characterising
/// property and needs no limits.
theorem tail_sup_mono(a: Nat -> Real, b: Nat -> Real, n: Nat) {
    is_bounded_above(a) and is_bounded_above(b)
        and (forall(k: Nat) { a(k) <= b(k) })
        implies tail_sup(a, n) <= tail_sup(b, n)
} by {
    if is_bounded_above(a) and is_bounded_above(b)
        and forall(k: Nat) { a(k) <= b(k) } {
        forall(k: Nat) {
            if n <= k {
                (a(k) <= b(k))
                tail_sup_bounds_terms(b, n, k)
                b(k) <= tail_sup(b, n)
                lte_trans(a(k), b(k), tail_sup(b, n))
                a(k) <= tail_sup(b, n)
            }
            (n <= k implies a(k) <= tail_sup(b, n))
        }
        tail_sup_is_least(a, n, tail_sup(b, n))
        tail_sup(a, n) <= tail_sup(b, n)
    }
}

/// A tail infimum is monotone in the sequence.
theorem tail_inf_mono(a: Nat -> Real, b: Nat -> Real, n: Nat) {
    is_bounded_below(a) and is_bounded_below(b)
        and (forall(k: Nat) { a(k) <= b(k) })
        implies tail_inf(a, n) <= tail_inf(b, n)
} by {
    if is_bounded_below(a) and is_bounded_below(b)
        and forall(k: Nat) { a(k) <= b(k) } {
        forall(k: Nat) {
            if n <= k {
                tail_inf_bounds_terms(a, n, k)
                tail_inf(a, n) <= a(k)
                (a(k) <= b(k))
                lte_trans(tail_inf(a, n), a(k), b(k))
                tail_inf(a, n) <= b(k)
            }
            (n <= k implies tail_inf(a, n) <= b(k))
        }
        tail_inf_is_greatest(b, n, tail_inf(a, n))
        tail_inf(a, n) <= tail_inf(b, n)
    }
}

/// The limit superior is monotone in the sequence.
///
/// The tail suprema are monotone, so the negated ones are antitone, and a termwise comparison of
/// convergent sequences is inherited by their limits.
theorem limsup_mono(a: Nat -> Real, b: Nat -> Real) {
    is_bounded_above(a) and is_bounded_below(a) and is_bounded_above(b)
        and is_bounded_below(b) and (forall(k: Nat) { a(k) <= b(k) })
        implies limsup(a) <= limsup(b)
} by {
    if is_bounded_above(a) and is_bounded_below(a) and is_bounded_above(b)
        and is_bounded_below(b) and forall(k: Nat) { a(k) <= b(k) } {
        neg_tail_sup_converges(a)
        converges(neg_tail_sup(a))
        neg_tail_sup_converges(b)
        converges(neg_tail_sup(b))
        forall(n: Nat) {
            tail_sup_mono(a, b, n)
            tail_sup(a, n) <= tail_sup(b, n)
            neg_reverses_lte(tail_sup(a, n), tail_sup(b, n))
            -tail_sup(b, n) <= -tail_sup(a, n)
            (neg_tail_sup(b)(n) = -tail_sup(b, n))
            (neg_tail_sup(a)(n) = -tail_sup(a, n))
            neg_tail_sup(b)(n) <= neg_tail_sup(a)(n)
        }
        (seq_lte(neg_tail_sup(b), neg_tail_sup(a)) = forall(n: Nat) {
            neg_tail_sup(b)(n) <= neg_tail_sup(a)(n)
        })
        seq_lte(neg_tail_sup(b), neg_tail_sup(a))
        seq_lte_preserves_limit(neg_tail_sup(b), neg_tail_sup(a))
        limit(neg_tail_sup(b)) <= limit(neg_tail_sup(a))
        neg_reverses_lte(limit(neg_tail_sup(b)), limit(neg_tail_sup(a)))
        -limit(neg_tail_sup(a)) <= -limit(neg_tail_sup(b))
        (limsup(a) = -limit(neg_tail_sup(a)))
        (limsup(b) = -limit(neg_tail_sup(b)))
        limsup(a) <= limsup(b)
    }
}

/// The upper density is monotone in the predicate.
///
/// A stronger predicate has a smaller density sequence at every index, hence a smaller limit
/// superior.
theorem upper_density_mono(p: Nat -> Bool, r: Nat -> Bool) {
    (forall(i: Nat) { p(i) implies r(i) })
        implies upper_density(p) <= upper_density(r)
} by {
    if forall(i: Nat) { p(i) implies r(i) } {
        density_seq_bounded_above(p)
        is_bounded_above(density_seq(p))
        density_seq_bounded_below(p)
        is_bounded_below(density_seq(p))
        density_seq_bounded_above(r)
        is_bounded_above(density_seq(r))
        density_seq_bounded_below(r)
        is_bounded_below(density_seq(r))
        forall(k: Nat) {
            density_seq_mono(p, r, k)
            density_seq(p, k) <= density_seq(r, k)
            (density_seq(p)(k) = density_seq(p, k))
            (density_seq(r)(k) = density_seq(r, k))
            density_seq(p)(k) <= density_seq(r)(k)
        }
        limsup_mono(density_seq(p), density_seq(r))
        limsup(density_seq(p)) <= limsup(density_seq(r))
        (upper_density(p) = limsup(density_seq(p)))
        (upper_density(r) = limsup(density_seq(r)))
        upper_density(p) <= upper_density(r)
    }
}

/// The limit inferior is monotone in the sequence.
///
/// The tail infima are monotone and increase to the limit inferior, so the comparison passes to
/// the limits directly.
theorem liminf_mono(a: Nat -> Real, b: Nat -> Real) {
    is_bounded_above(a) and is_bounded_below(a) and is_bounded_above(b)
        and is_bounded_below(b) and (forall(k: Nat) { a(k) <= b(k) })
        implies liminf(a) <= liminf(b)
} by {
    if is_bounded_above(a) and is_bounded_below(a) and is_bounded_above(b)
        and is_bounded_below(b) and forall(k: Nat) { a(k) <= b(k) } {
        neg_seq_bounded_above(a)
        is_bounded_above(neg_seq(a))
        neg_seq_bounded_below(a)
        is_bounded_below(neg_seq(a))
        neg_seq_bounded_above(b)
        is_bounded_above(neg_seq(b))
        neg_seq_bounded_below(b)
        is_bounded_below(neg_seq(b))
        neg_tail_sup_converges(neg_seq(a))
        converges(neg_tail_sup(neg_seq(a)))
        neg_tail_sup_converges(neg_seq(b))
        converges(neg_tail_sup(neg_seq(b)))
        forall(n: Nat) {
            tail_inf_mono(a, b, n)
            tail_inf(a, n) <= tail_inf(b, n)
            neg_tail_sup_neg_seq_eq_tail_inf(a, n)
            neg_tail_sup(neg_seq(a))(n) = tail_inf(a, n)
            neg_tail_sup_neg_seq_eq_tail_inf(b, n)
            neg_tail_sup(neg_seq(b))(n) = tail_inf(b, n)
            neg_tail_sup(neg_seq(a))(n) <= neg_tail_sup(neg_seq(b))(n)
        }
        (seq_lte(neg_tail_sup(neg_seq(a)), neg_tail_sup(neg_seq(b))) = forall(n: Nat) {
            neg_tail_sup(neg_seq(a))(n) <= neg_tail_sup(neg_seq(b))(n)
        })
        seq_lte(neg_tail_sup(neg_seq(a)), neg_tail_sup(neg_seq(b)))
        seq_lte_preserves_limit(neg_tail_sup(neg_seq(a)), neg_tail_sup(neg_seq(b)))
        limit(neg_tail_sup(neg_seq(a))) <= limit(neg_tail_sup(neg_seq(b)))
        liminf_eq_limit_tail_inf(a)
        liminf(a) = limit(neg_tail_sup(neg_seq(a)))
        liminf_eq_limit_tail_inf(b)
        liminf(b) = limit(neg_tail_sup(neg_seq(b)))
        liminf(a) <= liminf(b)
    }
}

/// The lower density is monotone in the predicate.
theorem lower_density_mono(p: Nat -> Bool, r: Nat -> Bool) {
    (forall(i: Nat) { p(i) implies r(i) })
        implies lower_density(p) <= lower_density(r)
} by {
    if forall(i: Nat) { p(i) implies r(i) } {
        density_seq_bounded_above(p)
        is_bounded_above(density_seq(p))
        density_seq_bounded_below(p)
        is_bounded_below(density_seq(p))
        density_seq_bounded_above(r)
        is_bounded_above(density_seq(r))
        density_seq_bounded_below(r)
        is_bounded_below(density_seq(r))
        forall(k: Nat) {
            density_seq_mono(p, r, k)
            density_seq(p, k) <= density_seq(r, k)
            (density_seq(p)(k) = density_seq(p, k))
            (density_seq(r)(k) = density_seq(r, k))
            density_seq(p)(k) <= density_seq(r)(k)
        }
        liminf_mono(density_seq(p), density_seq(r))
        liminf(density_seq(p)) <= liminf(density_seq(r))
        (lower_density(p) = liminf(density_seq(p)))
        (lower_density(r) = liminf(density_seq(r)))
        lower_density(p) <= lower_density(r)
    }
}
