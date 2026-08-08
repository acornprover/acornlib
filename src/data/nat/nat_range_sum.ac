from nat import Nat, sum_lte
from algebra.add_comm_monoid import AddCommMonoid
from algebra.add_semigroup import add_fn
from algebra.add_comm_monoid_rearrange import add_swap_inner, add_assoc_right

numerals Nat

/// The sum of `f(0), ..., f(n - 1)`.
///
/// Indexed by an initial range of the naturals rather than by a finite set, so the recurrence
/// is available directly and induction on the upper limit needs no set machinery.
define range_sum[A: AddCommMonoid](f: Nat -> A, n: Nat) -> A {
    match n {
        Nat.zero {
            A.0
        }
        Nat.suc(k) {
            range_sum(f, k) + f(k)
        }
    }
}

/// The empty range sums to zero.
theorem range_sum_zero[A: AddCommMonoid](f: Nat -> A) {
    range_sum(f, Nat.0) = A.0
}

/// The standard recurrence: one more term is added at the top.
theorem range_sum_suc[A: AddCommMonoid](f: Nat -> A, k: Nat) {
    range_sum(f, k.suc) = range_sum(f, k) + f(k)
}

/// A range of one term is that term.
theorem range_sum_one[A: AddCommMonoid](f: Nat -> A) {
    range_sum(f, Nat.1) = f(Nat.0)
} by {
    range_sum_suc(f, Nat.0)
    range_sum(f, Nat.1) = range_sum(f, Nat.0) + f(Nat.0)
    range_sum_zero(f)
    range_sum(f, Nat.1) = A.0 + f(Nat.0)
    A.0 + f(Nat.0) = f(Nat.0)
}

/// Summands agreeing below the limit give equal sums.
///
/// The sum depends on the function only through its values on the range, which is what lets a
/// summand be replaced by any convenient description of it.
theorem range_sum_congr[A: AddCommMonoid](f: Nat -> A, g: Nat -> A, n: Nat) {
    (forall(i: Nat) { i < n implies f(i) = g(i) }) implies range_sum(f, n) = range_sum(g, n)
} by {
    define p(x: Nat) -> Bool {
        (forall(i: Nat) { i < x implies f(i) = g(i) }) implies range_sum(f, x) = range_sum(g, x)
    }
    range_sum_zero(f)
    range_sum_zero(g)
    range_sum(f, Nat.0) = range_sum(g, Nat.0)
    p(Nat.0)
    forall(k: Nat) {
        if p(k) {
            if forall(i: Nat) { i < k.suc implies f(i) = g(i) } {
                forall(i: Nat) {
                    if i < k {
                        i < k.suc
                        f(i) = g(i)
                    }
                    (i < k implies f(i) = g(i))
                }
                range_sum(f, k) = range_sum(g, k)
                k < k.suc
                f(k) = g(k)
                range_sum_suc(f, k)
                range_sum_suc(g, k)
                range_sum(f, k.suc) = range_sum(g, k.suc)
            }
            p(k.suc)
        }
        (p(k) implies p(k.suc))
    }
    p(Nat.0) and forall(k: Nat) {
        p(k) implies p(k.suc)
    }
    Nat.induction(p)
    p(n)
}

/// A range sum of a pointwise sum splits.
theorem range_sum_add[A: AddCommMonoid](f: Nat -> A, g: Nat -> A, n: Nat) {
    range_sum(add_fn(f, g), n) = range_sum(f, n) + range_sum(g, n)
} by {
    define p(x: Nat) -> Bool {
        range_sum(add_fn(f, g), x) = range_sum(f, x) + range_sum(g, x)
    }
    range_sum_zero(add_fn(f, g))
    range_sum_zero(f)
    range_sum_zero(g)
    A.0 + A.0 = A.0
    range_sum(add_fn(f, g), Nat.0) = range_sum(f, Nat.0) + range_sum(g, Nat.0)
    p(Nat.0)
    forall(k: Nat) {
        if p(k) {
            range_sum(add_fn(f, g), k) = range_sum(f, k) + range_sum(g, k)
            range_sum_suc(add_fn(f, g), k)
            range_sum(add_fn(f, g), k.suc) = range_sum(add_fn(f, g), k) + add_fn(f, g, k)
            add_fn(f, g, k) = f(k) + g(k)
            (range_sum(add_fn(f, g), k.suc) =
                (range_sum(f, k) + range_sum(g, k)) + (f(k) + g(k)))
            add_swap_inner(range_sum(f, k), range_sum(g, k), f(k), g(k))
            ((range_sum(f, k) + range_sum(g, k)) + (f(k) + g(k)) =
                (range_sum(f, k) + f(k)) + (range_sum(g, k) + g(k)))
            range_sum_suc(f, k)
            range_sum_suc(g, k)
            range_sum(add_fn(f, g), k.suc) = range_sum(f, k.suc) + range_sum(g, k.suc)
            p(k.suc)
        }
        (p(k) implies p(k.suc))
    }
    p(Nat.0) and forall(k: Nat) {
        p(k) implies p(k.suc)
    }
    Nat.induction(p)
    p(n)
}

/// The summand shifted so that index `i` reads the value at `a + i`.
define shift_fn[A: AddCommMonoid](f: Nat -> A, a: Nat, i: Nat) -> A {
    f(a + i)
}

/// A range sum splits at an arbitrary midpoint.
///
/// The upper part is a range sum of the shifted summand, so the identity stays within range
/// sums rather than needing a notion of a sum over an interval.
theorem range_sum_split[A: AddCommMonoid](f: Nat -> A, m: Nat, n: Nat) {
    range_sum(f, m + n) = range_sum(f, m) + range_sum(shift_fn(f, m), n)
} by {
    define p(x: Nat) -> Bool {
        range_sum(f, m + x) = range_sum(f, m) + range_sum(shift_fn(f, m), x)
    }
    m + Nat.0 = m
    range_sum_zero(shift_fn(f, m))
    range_sum(f, m) + A.0 = range_sum(f, m)
    range_sum(f, m + Nat.0) = range_sum(f, m) + range_sum(shift_fn(f, m), Nat.0)
    p(Nat.0)
    forall(k: Nat) {
        if p(k) {
            range_sum(f, m + k) = range_sum(f, m) + range_sum(shift_fn(f, m), k)
            m + k.suc = (m + k).suc
            range_sum_suc(f, m + k)
            range_sum(f, (m + k).suc) = range_sum(f, m + k) + f(m + k)
            range_sum_suc(shift_fn(f, m), k)
            range_sum(shift_fn(f, m), k.suc) = range_sum(shift_fn(f, m), k) + shift_fn(f, m, k)
            shift_fn(f, m, k) = f(m + k)
            range_sum(shift_fn(f, m), k.suc) = range_sum(shift_fn(f, m), k) + f(m + k)
            (range_sum(f, m + k.suc) =
                (range_sum(f, m) + range_sum(shift_fn(f, m), k)) + f(m + k))
            add_assoc_right(range_sum(f, m), range_sum(shift_fn(f, m), k), f(m + k))
            ((range_sum(f, m) + range_sum(shift_fn(f, m), k)) + f(m + k) =
                range_sum(f, m) + (range_sum(shift_fn(f, m), k) + f(m + k)))
            range_sum(f, m + k.suc) = range_sum(f, m) + range_sum(shift_fn(f, m), k.suc)
            p(k.suc)
        }
        (p(k) implies p(k.suc))
    }
    p(Nat.0) and forall(k: Nat) {
        p(k) implies p(k.suc)
    }
    Nat.induction(p)
    p(n)
}

/// Shifting by zero changes nothing.
theorem shift_fn_zero[A: AddCommMonoid](f: Nat -> A, i: Nat) {
    shift_fn(f, Nat.0, i) = f(i)
} by {
    shift_fn(f, Nat.0, i) = f(Nat.0 + i)
    Nat.0 + i = i
}

/// Shifting twice is shifting by the total.
theorem shift_fn_add[A: AddCommMonoid](f: Nat -> A, a: Nat, b: Nat, i: Nat) {
    shift_fn(shift_fn(f, a), b, i) = shift_fn(f, a + b, i)
} by {
    shift_fn(shift_fn(f, a), b, i) = shift_fn(f, a)(b + i)
    shift_fn(f, a)(b + i) = f(a + (b + i))
    a + (b + i) = (a + b) + i
    shift_fn(f, a + b, i) = f((a + b) + i)
}

/// A range sum of a shifted summand is the sum over the shifted range.
///
/// The reindexing identity, stated so that the offset appears once on each side.
theorem range_sum_shift_split[A: AddCommMonoid](f: Nat -> A, a: Nat, b: Nat, n: Nat) {
    range_sum(shift_fn(f, a + b), n) = range_sum(shift_fn(shift_fn(f, a), b), n)
} by {
    forall(i: Nat) {
        shift_fn_add(f, a, b, i)
        shift_fn(shift_fn(f, a), b, i) = shift_fn(f, a + b, i)
        (i < n implies shift_fn(f, a + b, i) = shift_fn(shift_fn(f, a), b, i))
    }
    range_sum_congr(shift_fn(f, a + b), shift_fn(shift_fn(f, a), b), n)
    range_sum(shift_fn(f, a + b), n) = range_sum(shift_fn(shift_fn(f, a), b), n)
}

/// A range sum splits into three parts at two midpoints.
theorem range_sum_split_twice[A: AddCommMonoid](f: Nat -> A, a: Nat, b: Nat, c: Nat) {
    range_sum(f, a + (b + c)) = (range_sum(f, a) + range_sum(shift_fn(f, a), b))
        + range_sum(shift_fn(shift_fn(f, a), b), c)
} by {
    range_sum_split(f, a, b + c)
    range_sum(f, a + (b + c)) = range_sum(f, a) + range_sum(shift_fn(f, a), b + c)
    range_sum_split(shift_fn(f, a), b, c)
    (range_sum(shift_fn(f, a), b + c) =
        range_sum(shift_fn(f, a), b) + range_sum(shift_fn(shift_fn(f, a), b), c))
    (range_sum(f, a + (b + c)) = range_sum(f, a)
        + (range_sum(shift_fn(f, a), b) + range_sum(shift_fn(shift_fn(f, a), b), c)))
    add_assoc_right(range_sum(f, a), range_sum(shift_fn(f, a), b),
        range_sum(shift_fn(shift_fn(f, a), b), c))
    (range_sum(f, a) + (range_sum(shift_fn(f, a), b)
        + range_sum(shift_fn(shift_fn(f, a), b), c)) =
        (range_sum(f, a) + range_sum(shift_fn(f, a), b))
            + range_sum(shift_fn(shift_fn(f, a), b), c))
}

/// A range sum of the zero summand is zero.
define zero_fn[A: AddCommMonoid](i: Nat) -> A {
    A.0
}

/// The zero summand sums to zero over any range.
theorem range_sum_zero_fn[A: AddCommMonoid](n: Nat) {
    range_sum(zero_fn[A], n) = A.0
} by {
    define p(x: Nat) -> Bool {
        range_sum(zero_fn[A], x) = A.0
    }
    range_sum_zero(zero_fn[A])
    p(Nat.0)
    forall(k: Nat) {
        if p(k) {
            range_sum(zero_fn[A], k) = A.0
            range_sum_suc(zero_fn[A], k)
            range_sum(zero_fn[A], k.suc) = range_sum(zero_fn[A], k) + zero_fn[A](k)
            zero_fn[A](k) = A.0
            range_sum(zero_fn[A], k.suc) = A.0 + A.0
            A.0 + A.0 = A.0
            range_sum(zero_fn[A], k.suc) = A.0
            p(k.suc)
        }
        (p(k) implies p(k.suc))
    }
    p(Nat.0) and forall(k: Nat) {
        p(k) implies p(k.suc)
    }
    Nat.induction(p)
    p(n)
}

/// Range sums of naturals are monotone in the summand.
///
/// Stated over the naturals rather than an ordered semiring, since `AddCommMonoid` carries no
/// order and the ordered-semiring layer has no range sums yet.
theorem range_sum_le(f: Nat -> Nat, g: Nat -> Nat, n: Nat) {
    (forall(i: Nat) { f(i) <= g(i) }) implies range_sum(f, n) <= range_sum(g, n)
} by {
    if forall(i: Nat) { f(i) <= g(i) } {
        define p(x: Nat) -> Bool {
            range_sum(f, x) <= range_sum(g, x)
        }
        range_sum_zero(f)
        range_sum_zero(g)
        Nat.0 <= Nat.0
        range_sum(f, Nat.0) <= range_sum(g, Nat.0)
        p(Nat.0)
        forall(k: Nat) {
            if p(k) {
                range_sum(f, k) <= range_sum(g, k)
                f(k) <= g(k)
                sum_lte(range_sum(f, k), f(k), range_sum(g, k), g(k))
                range_sum(f, k) + f(k) <= range_sum(g, k) + g(k)
                range_sum_suc(f, k)
                range_sum_suc(g, k)
                range_sum(f, k.suc) <= range_sum(g, k.suc)
                p(k.suc)
            }
            (p(k) implies p(k.suc))
        }
        p(Nat.0) and forall(k: Nat) {
            p(k) implies p(k.suc)
        }
        Nat.induction(p)
        p(n)
        range_sum(f, n) <= range_sum(g, n)
    }
}

/// A range sum of naturals is bounded by the count times a bound on the terms.
///
/// The standard crude estimate: every term is at most `c`, so the total is at most `n * c`.
theorem range_sum_le_count_mul(f: Nat -> Nat, c: Nat, n: Nat) {
    (forall(i: Nat) { f(i) <= c }) implies range_sum(f, n) <= n * c
} by {
    if forall(i: Nat) { f(i) <= c } {
        define p(x: Nat) -> Bool {
            range_sum(f, x) <= x * c
        }
        range_sum_zero(f)
        Nat.0 * c = Nat.0
        range_sum(f, Nat.0) <= Nat.0 * c
        p(Nat.0)
        forall(k: Nat) {
            if p(k) {
                range_sum(f, k) <= k * c
                f(k) <= c
                sum_lte(range_sum(f, k), f(k), k * c, c)
                range_sum(f, k) + f(k) <= k * c + c
                range_sum_suc(f, k)
                range_sum(f, k.suc) <= k * c + c
                k.suc = k + Nat.1
                (k + Nat.1) * c = k * c + Nat.1 * c
                Nat.1 * c = c
                k.suc * c = k * c + c
                range_sum(f, k.suc) <= k.suc * c
                p(k.suc)
            }
            (p(k) implies p(k.suc))
        }
        p(Nat.0) and forall(k: Nat) {
            p(k) implies p(k.suc)
        }
        Nat.induction(p)
        p(n)
        range_sum(f, n) <= n * c
    }
}
