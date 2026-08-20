from nat import Nat
from real import Real
from data.nat.nat_density import density_seq
from data.nat.nat_density_complement import not_pred
from analysis import is_bounded_above, limsup, is_bounded_below, liminf, one_minus_seq,
    liminf_one_minus, one_minus_one_minus, add_one_minus
from data.basic.functions import function_extensionality
from data.nat.nat_density_complement import not_pred_apply
from data.nat.nat_density_extremes import upper_density, lower_density, density_seq_bounded_above,
    density_seq_bounded_below
from data.nat.nat_density_complement_extremes import density_seq_complement_eq

/// Complementing twice restores the predicate.
///
/// The two truth values have to be tried by hand: proof search does not case split on a
/// boolean, so the double negation stays unresolved until both cases are written out.
theorem not_pred_involution(p: Nat -> Bool) {
    not_pred(not_pred(p)) = p
} by {
    forall(i: Nat) {
        not_pred_apply(p, i)
        (not_pred(p)(i) = not p(i))
        not_pred_apply(not_pred(p), i)
        (not_pred(not_pred(p))(i) = not not_pred(p)(i))
        if p(i) {
            not not_pred(p)(i)
            not_pred(not_pred(p))(i)
            (not_pred(not_pred(p))(i) = p(i))
        }
        if not p(i) {
            not_pred(p)(i)
            not not_pred(not_pred(p))(i)
            (not_pred(not_pred(p))(i) = p(i))
        }
        (not_pred(not_pred(p))(i) = p(i))
    }
    function_extensionality[Nat, Bool](not_pred(not_pred(p)), p)
    not_pred(not_pred(p)) = p
}

/// The lower density of a complement is one minus the upper density.
///
/// The dual of `upper_density_complement`. Between them the two one-sided densities of a
/// complement are determined by those of the predicate, with the roles exchanged.
theorem lower_density_complement(p: Nat -> Bool) {
    lower_density(not_pred(p)) = Real.1 - upper_density(p)
} by {
    density_seq_bounded_above(p)
    is_bounded_above(density_seq(p))
    density_seq_bounded_below(p)
    is_bounded_below(density_seq(p))
    liminf_one_minus(density_seq(p))
    liminf(one_minus_seq(density_seq(p))) = Real.1 - limsup(density_seq(p))
    density_seq_complement_eq(p)
    density_seq(not_pred(p)) = one_minus_seq(density_seq(p))
    liminf(density_seq(not_pred(p))) = Real.1 - limsup(density_seq(p))
    (lower_density(not_pred(p)) = liminf(density_seq(not_pred(p))))
    (upper_density(p) = limsup(density_seq(p)))
    lower_density(not_pred(p)) = Real.1 - upper_density(p)
}

/// The upper density of a predicate and the lower density of its complement add to one.
theorem upper_density_add_lower_density_complement(p: Nat -> Bool) {
    upper_density(p) + lower_density(not_pred(p)) = Real.1
} by {
    lower_density_complement(p)
    lower_density(not_pred(p)) = Real.1 - upper_density(p)
    add_one_minus(upper_density(p))
    (upper_density(p) + (Real.1 - upper_density(p)) = Real.1)
    upper_density(p) + lower_density(not_pred(p)) = Real.1
}

/// A predicate has upper density one exactly when its complement has lower density zero.
///
/// The form the complement identity is usually used in: a set is dense along some subsequence
/// of ranges exactly when its complement thins out along that same subsequence.
theorem upper_density_one_iff_complement_lower_zero(p: Nat -> Bool) {
    (upper_density(p) = Real.1) = (lower_density(not_pred(p)) = Real.0)
} by {
    lower_density_complement(p)
    lower_density(not_pred(p)) = Real.1 - upper_density(p)
    if upper_density(p) = Real.1 {
        (Real.1 - Real.1 = Real.0)
        lower_density(not_pred(p)) = Real.0
    }
    if lower_density(not_pred(p)) = Real.0 {
        Real.1 - upper_density(p) = Real.0
        one_minus_one_minus(upper_density(p))
        (Real.1 - (Real.1 - upper_density(p)) = upper_density(p))
        (Real.1 - Real.0 = Real.1)
        upper_density(p) = Real.1
    }
    (upper_density(p) = Real.1) = (lower_density(not_pred(p)) = Real.0)
}

/// A predicate has lower density one exactly when its complement has upper density zero.
theorem lower_density_one_iff_complement_upper_zero(p: Nat -> Bool) {
    (lower_density(p) = Real.1) = (upper_density(not_pred(p)) = Real.0)
} by {
    lower_density_complement(not_pred(p))
    not_pred_involution(p)
    (not_pred(not_pred(p)) = p)
    lower_density(p) = Real.1 - upper_density(not_pred(p))
    if lower_density(p) = Real.1 {
        one_minus_one_minus(upper_density(not_pred(p)))
        (Real.1 - (Real.1 - upper_density(not_pred(p))) = upper_density(not_pred(p)))
        (Real.1 - Real.1 = Real.0)
        upper_density(not_pred(p)) = Real.0
    }
    if upper_density(not_pred(p)) = Real.0 {
        (Real.1 - Real.0 = Real.1)
        lower_density(p) = Real.1
    }
    (lower_density(p) = Real.1) = (upper_density(not_pred(p)) = Real.0)
}
