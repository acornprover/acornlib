from nat import Nat, lte_trans, divides_trans, has_prime_divisor,
    gcd_divides_left, gcd_divides_right, gcd_nonzero_left

numerals Nat

/// True if every prime factor of `n` is at most `b`.
///
/// Stated over all primes dividing `n` rather than over a factorisation, so it needs no
/// list machinery and composes directly with divisibility facts.
define is_smooth(n: Nat, b: Nat) -> Bool {
    forall(p: Nat) {
        p.is_prime and p.divides(n) implies p <= b
    }
}

/// A prime factor of a smooth number is below the bound.
theorem is_smooth_apply(n: Nat, b: Nat, p: Nat) {
    is_smooth(n, b) and p.is_prime and p.divides(n) implies p <= b
} by {
    if is_smooth(n, b) and p.is_prime and p.divides(n) {
        is_smooth(n, b) = forall(q: Nat) {
            q.is_prime and q.divides(n) implies q <= b
        }
        forall(q: Nat) {
            q.is_prime and q.divides(n) implies q <= b
        }
        p.is_prime and p.divides(n) implies p <= b
        p <= b
    }
}

/// A pointwise bound on prime factors is smoothness.
theorem is_smooth_intro(n: Nat, b: Nat) {
    (forall(p: Nat) { p.is_prime and p.divides(n) implies p <= b }) implies is_smooth(n, b)
} by {
    if forall(p: Nat) { p.is_prime and p.divides(n) implies p <= b } {
        is_smooth(n, b) = forall(q: Nat) {
            q.is_prime and q.divides(n) implies q <= b
        }
        is_smooth(n, b)
    }
}

/// Smoothness weakens as the bound grows.
theorem is_smooth_weaken(n: Nat, b: Nat, c: Nat) {
    is_smooth(n, b) and b <= c implies is_smooth(n, c)
} by {
    if is_smooth(n, b) and b <= c {
        forall(p: Nat) {
            if p.is_prime and p.divides(n) {
                is_smooth_apply(n, b, p)
                p <= b
                lte_trans(p, b, c)
                p <= c
            }
            p.is_prime and p.divides(n) implies p <= c
        }
        is_smooth_intro(n, c)
        is_smooth(n, c)
    }
}

/// Smoothness passes to divisors.
///
/// A prime dividing a divisor divides the original, so it obeys the same bound.
theorem is_smooth_of_divides(n: Nat, m: Nat, b: Nat) {
    is_smooth(n, b) and m.divides(n) implies is_smooth(m, b)
} by {
    if is_smooth(n, b) and m.divides(n) {
        forall(p: Nat) {
            if p.is_prime and p.divides(m) {
                divides_trans(p, m, n)
                p.divides(n)
                is_smooth_apply(n, b, p)
                p <= b
            }
            p.is_prime and p.divides(m) implies p <= b
        }
        is_smooth_intro(m, b)
        is_smooth(m, b)
    }
}

/// True if every prime factor of `n` is greater than `b`.
define is_rough(n: Nat, b: Nat) -> Bool {
    forall(p: Nat) {
        p.is_prime and p.divides(n) implies b < p
    }
}

/// A prime factor of a rough number is above the bound.
theorem is_rough_apply(n: Nat, b: Nat, p: Nat) {
    is_rough(n, b) and p.is_prime and p.divides(n) implies b < p
} by {
    if is_rough(n, b) and p.is_prime and p.divides(n) {
        is_rough(n, b) = forall(q: Nat) {
            q.is_prime and q.divides(n) implies b < q
        }
        forall(q: Nat) {
            q.is_prime and q.divides(n) implies b < q
        }
        p.is_prime and p.divides(n) implies b < p
        b < p
    }
}

/// A pointwise lower bound on prime factors is roughness.
theorem is_rough_intro(n: Nat, b: Nat) {
    (forall(p: Nat) { p.is_prime and p.divides(n) implies b < p }) implies is_rough(n, b)
} by {
    if forall(p: Nat) { p.is_prime and p.divides(n) implies b < p } {
        is_rough(n, b) = forall(q: Nat) {
            q.is_prime and q.divides(n) implies b < q
        }
        is_rough(n, b)
    }
}

/// Roughness passes to divisors.
theorem is_rough_of_divides(n: Nat, m: Nat, b: Nat) {
    is_rough(n, b) and m.divides(n) implies is_rough(m, b)
} by {
    if is_rough(n, b) and m.divides(n) {
        forall(p: Nat) {
            if p.is_prime and p.divides(m) {
                divides_trans(p, m, n)
                p.divides(n)
                is_rough_apply(n, b, p)
                b < p
            }
            p.is_prime and p.divides(m) implies b < p
        }
        is_rough_intro(m, b)
        is_rough(m, b)
    }
}

/// A common divisor of a smooth number and a rough number is one.
///
/// This is the separation that makes the two predicates useful together: the prime
/// factors of a `b`-smooth number and those of a `b`-rough number lie on opposite sides
/// of `b`, so no prime divides both.
theorem smooth_rough_common_divisor_is_one(m: Nat, n: Nat, b: Nat, d: Nat) {
    is_smooth(m, b) and is_rough(n, b) and d.divides(m) and d.divides(n) and d != Nat.0
        implies d = Nat.1
} by {
    if is_smooth(m, b) and is_rough(n, b) and d.divides(m) and d.divides(n) and d != Nat.0 {
        if d != Nat.1 {
            Nat.1 < d
            has_prime_divisor(d)
            let (p: Nat) satisfy {
                p.is_prime and p.divides(d)
            }
            divides_trans(p, d, m)
            p.divides(m)
            divides_trans(p, d, n)
            p.divides(n)
            is_smooth_apply(m, b, p)
            p <= b
            is_rough_apply(n, b, p)
            b < p
            false
        }
        d = Nat.1
    }
}

/// A nonzero smooth number is coprime to every rough number at the same bound.
theorem smooth_rough_coprime(m: Nat, n: Nat, b: Nat) {
    is_smooth(m, b) and is_rough(n, b) and m != Nat.0 implies m.coprime(n)
} by {
    if is_smooth(m, b) and is_rough(n, b) and m != Nat.0 {
        gcd_nonzero_left(m, n)
        m.gcd(n) != Nat.0
        gcd_divides_left(m, n)
        m.gcd(n).divides(m)
        gcd_divides_right(m, n)
        m.gcd(n).divides(n)
        smooth_rough_common_divisor_is_one(m, n, b, m.gcd(n))
        m.gcd(n) = Nat.1
        m.coprime(n) = (m.gcd(n) = Nat.1)
        m.coprime(n)
    }
}

/// A number both smooth and rough at the same bound has no prime factors.
///
/// No prime can be both at most `b` and greater than `b`.
theorem smooth_and_rough_no_prime_factor(n: Nat, b: Nat, p: Nat) {
    is_smooth(n, b) and is_rough(n, b) and p.is_prime implies not p.divides(n)
} by {
    if is_smooth(n, b) and is_rough(n, b) and p.is_prime {
        if p.divides(n) {
            is_smooth_apply(n, b, p)
            p <= b
            is_rough_apply(n, b, p)
            b < p
            false
        }
        not p.divides(n)
    }
}
