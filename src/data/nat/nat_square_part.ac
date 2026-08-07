from nat import Nat, divides_lte, divides_self, lte_and_lt, lte_trans, lte_antisymm,
    lte_mul_both, lt_mul_both, mul_cancel_left, mul_cancel_right, div_imp_mod,
    div_mod_decomp
from data.nat.nat_bounded_max import is_upper_bound_of, is_upper_bound_of_intro, is_max, is_max_apply,
    is_max_is_upper_bound, has_max
from data.nat.nat_square import is_square, is_square_intro, is_square_witness
from data.nat.nat_squarefree import is_squarefree, is_squarefree_apply, is_squarefree_intro

numerals Nat

/// True if the square of `d` divides `n`.
///
/// Packaged as a predicate on `Nat` so the maximisation of `src/nat_bounded_max.ac` applies.
define square_divisor_pred(n: Nat) -> (Nat -> Bool) {
    function(d: Nat) {
        (d * d).divides(n)
    }
}

/// One is always a square divisor.
theorem square_divisor_pred_one(n: Nat) {
    square_divisor_pred(n)(Nat.1)
} by {
    Nat.1 * Nat.1 = Nat.1
    Nat.1.divides(n)
    (Nat.1 * Nat.1).divides(n)
}

/// A square divisor of a positive number is no larger than it.
///
/// The divisor `d` is itself at most `d * d`, since a zero `d` would force `n` to be zero.
theorem square_divisor_pred_bound(n: Nat, d: Nat) {
    Nat.1 <= n and square_divisor_pred(n)(d) implies d <= n
} by {
    if Nat.1 <= n and square_divisor_pred(n)(d) {
        (d * d).divides(n)
        divides_lte(d * d, n)
        (n = Nat.0 or d * d <= n)
        if n = Nat.0 {
            Nat.1 <= Nat.0
            Nat.0 < Nat.1
            lte_and_lt(Nat.1, Nat.0, Nat.1)
            Nat.1 < Nat.1
            false
        }
        d * d <= n
        if d = Nat.0 {
            Nat.0 * Nat.0 = Nat.0
            Nat.0.divides(n)
            n = Nat.0
            false
        }
        Nat.1 <= d
        d * Nat.1 <= d * d
        d * Nat.1 = d
        d <= d * d
        lte_trans(d, d * d, n)
        d <= n
    }
}

/// The positive number itself bounds its square divisors.
theorem square_divisor_pred_upper_bound(n: Nat) {
    Nat.1 <= n implies is_upper_bound_of(square_divisor_pred(n), n)
} by {
    if Nat.1 <= n {
        forall(d: Nat) {
            if square_divisor_pred(n)(d) {
                square_divisor_pred_bound(n, d)
                d <= n
            }
            (square_divisor_pred(n)(d) implies d <= n)
        }
        is_upper_bound_of_intro(square_divisor_pred(n), n)
        is_upper_bound_of(square_divisor_pred(n), n)
    }
}

/// The largest number whose square divides `n`.
///
/// Defined by maximisation rather than through prime valuations, so it needs no factorisation
/// machinery. The family is nonempty, containing one, and bounded by `n`. At `n = 0` every
/// square divides, so the maximum does not exist and the defining property is stated only for
/// positive `n`.
let square_part(n: Nat) -> result: Nat satisfy {
    Nat.1 <= n implies is_max(square_divisor_pred(n), result)
} by {
    if Nat.1 <= n {
        square_divisor_pred_one(n)
        square_divisor_pred_upper_bound(n)
        is_upper_bound_of(square_divisor_pred(n), n)
        has_max(square_divisor_pred(n), Nat.1, n)
    }
}

/// The square of the square part divides the number.
theorem square_part_divides(n: Nat) {
    Nat.1 <= n implies (square_part(n) * square_part(n)).divides(n)
} by {
    if Nat.1 <= n {
        is_max(square_divisor_pred(n), square_part(n))
        is_max_apply(square_divisor_pred(n), square_part(n))
        square_divisor_pred(n)(square_part(n))
        (square_part(n) * square_part(n)).divides(n)
    }
}

/// No square divisor exceeds the square part.
theorem square_part_is_greatest(n: Nat, d: Nat) {
    Nat.1 <= n and (d * d).divides(n) implies d <= square_part(n)
} by {
    if Nat.1 <= n and (d * d).divides(n) {
        square_divisor_pred(n)(d)
        is_max(square_divisor_pred(n), square_part(n))
        is_max_is_upper_bound(square_divisor_pred(n), square_part(n), d)
        d <= square_part(n)
    }
}

/// The square part of a positive number is positive.
theorem square_part_pos(n: Nat) {
    Nat.1 <= n implies Nat.1 <= square_part(n)
} by {
    if Nat.1 <= n {
        Nat.1 * Nat.1 = Nat.1
        Nat.1.divides(n)
        (Nat.1 * Nat.1).divides(n)
        square_part_is_greatest(n, Nat.1)
        Nat.1 <= square_part(n)
    }
}

/// The cofactor left after dividing out the largest square.
///
/// Written with division rather than as a chosen cofactor so that it is total: the defining
/// equation below needs `n` positive, but the function itself does not.
define squarefree_part(n: Nat) -> Nat {
    n.div(square_part(n) * square_part(n))
}

/// Every positive number is its largest square times its squarefree part.
theorem square_squarefree_decomposition(n: Nat) {
    Nat.1 <= n implies square_part(n) * square_part(n) * squarefree_part(n) = n
} by {
    if Nat.1 <= n {
        square_part_divides(n)
        (square_part(n) * square_part(n)).divides(n)
        div_imp_mod(n, square_part(n) * square_part(n))
        n.mod(square_part(n) * square_part(n)) = Nat.0
        div_mod_decomp(n, square_part(n) * square_part(n))
        (n.div(square_part(n) * square_part(n)) * (square_part(n) * square_part(n))
            + n.mod(square_part(n) * square_part(n)) = n)
        (n.div(square_part(n) * square_part(n)) * (square_part(n) * square_part(n)) = n)
        (squarefree_part(n) * (square_part(n) * square_part(n)) = n)
        (square_part(n) * square_part(n) * squarefree_part(n) = n)
    }
}

/// The squarefree part of a positive number is positive.
theorem squarefree_part_pos(n: Nat) {
    Nat.1 <= n implies Nat.1 <= squarefree_part(n)
} by {
    if Nat.1 <= n {
        square_squarefree_decomposition(n)
        square_part(n) * square_part(n) * squarefree_part(n) = n
        if squarefree_part(n) = Nat.0 {
            square_part(n) * square_part(n) * Nat.0 = Nat.0
            n = Nat.0
            Nat.1 <= Nat.0
            Nat.0 < Nat.1
            lte_and_lt(Nat.1, Nat.0, Nat.1)
            Nat.1 < Nat.1
            false
        }
        Nat.1 <= squarefree_part(n)
    }
}

/// The squarefree part really is squarefree.
///
/// A square `c * c` dividing it would make `square_part(n) * c` a larger square divisor of `n`,
/// so `c` is one and the square is one. This is where the maximality of the square part is
/// used, and it is the whole content of the decomposition.
theorem squarefree_part_is_squarefree(n: Nat) {
    Nat.1 <= n implies is_squarefree(squarefree_part(n))
} by {
    if Nat.1 <= n {
        square_squarefree_decomposition(n)
        square_part(n) * square_part(n) * squarefree_part(n) = n
        square_part_pos(n)
        Nat.1 <= square_part(n)
        forall(e: Nat) {
            if is_square(e) and e.divides(squarefree_part(n)) {
                is_square_witness(e)
                exists(c: Nat) { c * c = e }
                let (c: Nat) satisfy {
                    c * c = e
                }
                (c * c).divides(squarefree_part(n))
                exists(t: Nat) { c * c * t = squarefree_part(n) }
                let (t: Nat) satisfy {
                    c * c * t = squarefree_part(n)
                }
                ((square_part(n) * c) * (square_part(n) * c) * t
                    = square_part(n) * square_part(n) * (c * c * t))
                ((square_part(n) * c) * (square_part(n) * c) * t
                    = square_part(n) * square_part(n) * squarefree_part(n))
                ((square_part(n) * c) * (square_part(n) * c) * t = n)
                exists(k: Nat) { (square_part(n) * c) * (square_part(n) * c) * k = n }
                ((square_part(n) * c) * (square_part(n) * c)).divides(n)
                square_part_is_greatest(n, square_part(n) * c)
                square_part(n) * c <= square_part(n)
                square_part(n) * c <= square_part(n) * Nat.1
                if Nat.2 <= c {
                    Nat.1 <= Nat.2
                    lte_trans(Nat.1, Nat.2, c)
                    Nat.1 <= c
                    lte_mul_both(square_part(n), Nat.1, c)
                    square_part(n) * Nat.1 <= square_part(n) * c
                    lte_antisymm(square_part(n) * c, square_part(n) * Nat.1)
                    square_part(n) * c = square_part(n) * Nat.1
                    if square_part(n) = Nat.0 {
                        Nat.1 <= Nat.0
                        Nat.0 < Nat.1
                        lte_and_lt(Nat.1, Nat.0, Nat.1)
                        Nat.1 < Nat.1
                        false
                    }
                    square_part(n) != Nat.0
                    mul_cancel_left(square_part(n), c, Nat.1)
                    c = Nat.1
                    Nat.2 <= Nat.1
                    Nat.1 < Nat.2
                    lte_and_lt(Nat.2, Nat.1, Nat.2)
                    Nat.2 < Nat.2
                    false
                }
                c < Nat.2
                c <= Nat.1
                if c = Nat.0 {
                    Nat.0 * Nat.0 = Nat.0
                    e = Nat.0
                    Nat.0.divides(squarefree_part(n))
                    squarefree_part(n) = Nat.0
                    squarefree_part_pos(n)
                    Nat.1 <= Nat.0
                    Nat.0 < Nat.1
                    lte_and_lt(Nat.1, Nat.0, Nat.1)
                    Nat.1 < Nat.1
                    false
                }
                Nat.1 <= c
                lte_antisymm(c, Nat.1)
                c = Nat.1
                Nat.1 * Nat.1 = Nat.1
                e = Nat.1
            }
            (is_square(e) and e.divides(squarefree_part(n)) implies e = Nat.1)
        }
        is_squarefree_intro(squarefree_part(n))
        is_squarefree(squarefree_part(n))
    }
}

/// Squaring reflects order.
///
/// The converse of monotonicity, which is what turns a divisibility between squares into a
/// comparison between their roots.
theorem square_lte_cancel(a: Nat, b: Nat) {
    a * a <= b * b implies a <= b
} by {
    if a * a <= b * b {
        if b < a {
            a != Nat.0
            lt_mul_both(a, b, a)
            a * b < a * a
            b <= a
            lte_mul_both(b, b, a)
            b * b <= b * a
            b * a = a * b
            b * b <= a * b
            lte_and_lt(b * b, a * b, a * a)
            b * b < a * a
            lte_and_lt(a * a, b * b, a * a)
            a * a < a * a
            false
        }
        a <= b
    }
}

/// The squarefree part divides the number.
theorem squarefree_part_divides(n: Nat) {
    Nat.1 <= n implies squarefree_part(n).divides(n)
} by {
    if Nat.1 <= n {
        square_squarefree_decomposition(n)
        square_part(n) * square_part(n) * squarefree_part(n) = n
        squarefree_part(n) * (square_part(n) * square_part(n)) = n
        exists(c: Nat) { squarefree_part(n) * c = n }
        squarefree_part(n).divides(n)
    }
}

/// A squarefree number has trivial square part and is its own squarefree part.
theorem squarefree_part_of_squarefree(n: Nat) {
    Nat.1 <= n and is_squarefree(n)
        implies square_part(n) = Nat.1 and squarefree_part(n) = n
} by {
    if Nat.1 <= n and is_squarefree(n) {
        square_part_divides(n)
        (square_part(n) * square_part(n)).divides(n)
        is_square_intro(square_part(n) * square_part(n), square_part(n))
        is_square(square_part(n) * square_part(n))
        is_squarefree_apply(n, square_part(n) * square_part(n))
        square_part(n) * square_part(n) = Nat.1
        square_part_pos(n)
        Nat.1 <= square_part(n)
        if Nat.2 <= square_part(n) {
            Nat.1 <= square_part(n)
            lte_mul_both(square_part(n), Nat.1, square_part(n))
            square_part(n) * Nat.1 <= square_part(n) * square_part(n)
            square_part(n) * Nat.1 = square_part(n)
            square_part(n) <= Nat.1
            lte_antisymm(square_part(n), Nat.1)
            square_part(n) = Nat.1
            Nat.2 <= Nat.1
            Nat.1 < Nat.2
            lte_and_lt(Nat.2, Nat.1, Nat.2)
            Nat.2 < Nat.2
            false
        }
        square_part(n) < Nat.2
        square_part(n) <= Nat.1
        lte_antisymm(square_part(n), Nat.1)
        square_part(n) = Nat.1
        square_squarefree_decomposition(n)
        Nat.1 * Nat.1 * squarefree_part(n) = n
        Nat.1 * Nat.1 = Nat.1
        Nat.1 * squarefree_part(n) = squarefree_part(n)
        squarefree_part(n) = n
        (square_part(n) = Nat.1 and squarefree_part(n) = n)
    }
}

/// A positive square has its root as square part and one as squarefree part.
theorem square_part_of_square(r: Nat) {
    Nat.1 <= r implies
        (square_part(r * r) = r and squarefree_part(r * r) = Nat.1)
} by {
    if Nat.1 <= r {
        lte_mul_both(r, Nat.1, r)
        r * Nat.1 <= r * r
        r * Nat.1 = r
        r <= r * r
        lte_trans(Nat.1, r, r * r)
        Nat.1 <= r * r
        divides_self(r * r)
        (r * r).divides(r * r)
        square_part_is_greatest(r * r, r)
        r <= square_part(r * r)
        square_part_divides(r * r)
        (square_part(r * r) * square_part(r * r)).divides(r * r)
        divides_lte(square_part(r * r) * square_part(r * r), r * r)
        (r * r = Nat.0 or square_part(r * r) * square_part(r * r) <= r * r)
        if r * r = Nat.0 {
            Nat.1 <= Nat.0
            Nat.0 < Nat.1
            lte_and_lt(Nat.1, Nat.0, Nat.1)
            Nat.1 < Nat.1
            false
        }
        square_part(r * r) * square_part(r * r) <= r * r
        square_lte_cancel(square_part(r * r), r)
        square_part(r * r) <= r
        lte_antisymm(square_part(r * r), r)
        square_part(r * r) = r
        square_squarefree_decomposition(r * r)
        r * r * squarefree_part(r * r) = r * r
        r * r != Nat.0
        r * r * squarefree_part(r * r) = r * r * Nat.1
        mul_cancel_left(r * r, squarefree_part(r * r), Nat.1)
        squarefree_part(r * r) = Nat.1
        (square_part(r * r) = r and squarefree_part(r * r) = Nat.1)
    }
}

/// A positive number is a square exactly when its squarefree part is one.
///
/// The decomposition read as a test: the square part carries everything, and what is left over
/// is the obstruction to being a square.
theorem is_square_iff_squarefree_part_one(n: Nat) {
    Nat.1 <= n implies (is_square(n) = (squarefree_part(n) = Nat.1))
} by {
    if Nat.1 <= n {
        if is_square(n) {
            is_square_witness(n)
            exists(r: Nat) { r * r = n }
            let (r: Nat) satisfy {
                r * r = n
            }
            if r = Nat.0 {
                Nat.0 * Nat.0 = Nat.0
                n = Nat.0
                Nat.1 <= Nat.0
                Nat.0 < Nat.1
                lte_and_lt(Nat.1, Nat.0, Nat.1)
                Nat.1 < Nat.1
                false
            }
            Nat.1 <= r
            square_part_of_square(r)
            squarefree_part(r * r) = Nat.1
            squarefree_part(n) = Nat.1
        }
        if squarefree_part(n) = Nat.1 {
            square_squarefree_decomposition(n)
            square_part(n) * square_part(n) * Nat.1 = n
            square_part(n) * square_part(n) = n
            is_square_intro(n, square_part(n))
            is_square(n)
        }
        (is_square(n) implies squarefree_part(n) = Nat.1)
        ((squarefree_part(n) = Nat.1) implies is_square(n))
        (is_square(n) = (squarefree_part(n) = Nat.1))
    }
}

/// The square part divides the number.
theorem square_part_divides_self(n: Nat) {
    Nat.1 <= n implies square_part(n).divides(n)
} by {
    if Nat.1 <= n {
        square_part_divides(n)
        (square_part(n) * square_part(n)).divides(n)
        exists(c: Nat) { square_part(n) * square_part(n) * c = n }
        let (c: Nat) satisfy {
            square_part(n) * square_part(n) * c = n
        }
        (square_part(n) * (square_part(n) * c) = n)
        exists(k: Nat) { square_part(n) * k = n }
        square_part(n).divides(n)
    }
}

/// A number with trivial square part is squarefree.
///
/// The converse of `squarefree_part_of_squarefree`. A square divisor `c * c` puts `c` below the
/// square part, which is one, so the divisor is one as well.
theorem squarefree_of_square_part_one(n: Nat) {
    Nat.1 <= n and square_part(n) = Nat.1 implies is_squarefree(n)
} by {
    if Nat.1 <= n and square_part(n) = Nat.1 {
        forall(e: Nat) {
            if is_square(e) and e.divides(n) {
                is_square_witness(e)
                exists(c: Nat) { c * c = e }
                let (c: Nat) satisfy {
                    c * c = e
                }
                (c * c).divides(n)
                square_part_is_greatest(n, c)
                c <= square_part(n)
                c <= Nat.1
                if c = Nat.0 {
                    Nat.0 * Nat.0 = Nat.0
                    e = Nat.0
                    Nat.0.divides(n)
                    n = Nat.0
                    Nat.1 <= Nat.0
                    Nat.0 < Nat.1
                    lte_and_lt(Nat.1, Nat.0, Nat.1)
                    Nat.1 < Nat.1
                    false
                }
                Nat.1 <= c
                lte_antisymm(c, Nat.1)
                c = Nat.1
                Nat.1 * Nat.1 = Nat.1
                e = Nat.1
            }
            (is_square(e) and e.divides(n) implies e = Nat.1)
        }
        is_squarefree_intro(n)
        is_squarefree(n)
    }
}

/// Squarefreeness of a positive number is exactly having trivial square part.
///
/// The decision form of the decomposition: `n` is squarefree precisely when the largest square
/// dividing it is one.
theorem is_squarefree_iff_square_part_one(n: Nat) {
    Nat.1 <= n implies (is_squarefree(n) = (square_part(n) = Nat.1))
} by {
    if Nat.1 <= n {
        if is_squarefree(n) {
            squarefree_part_of_squarefree(n)
            (square_part(n) = Nat.1 and squarefree_part(n) = n)
            square_part(n) = Nat.1
        }
        if square_part(n) = Nat.1 {
            squarefree_of_square_part_one(n)
            is_squarefree(n)
        }
        (is_squarefree(n) implies square_part(n) = Nat.1)
        ((square_part(n) = Nat.1) implies is_squarefree(n))
        (is_squarefree(n) = (square_part(n) = Nat.1))
    }
}

/// A positive number is squarefree exactly when it is its own squarefree part.
theorem is_squarefree_iff_squarefree_part_self(n: Nat) {
    Nat.1 <= n implies (is_squarefree(n) = (squarefree_part(n) = n))
} by {
    if Nat.1 <= n {
        if is_squarefree(n) {
            squarefree_part_of_squarefree(n)
            (square_part(n) = Nat.1 and squarefree_part(n) = n)
            squarefree_part(n) = n
        }
        if squarefree_part(n) = n {
            square_squarefree_decomposition(n)
            square_part(n) * square_part(n) * squarefree_part(n) = n
            square_part(n) * square_part(n) * n = n
            if n = Nat.0 {
                Nat.1 <= Nat.0
                Nat.0 < Nat.1
                lte_and_lt(Nat.1, Nat.0, Nat.1)
                Nat.1 < Nat.1
                false
            }
            n != Nat.0
            square_part(n) * square_part(n) * n = Nat.1 * n
            mul_cancel_right(n, square_part(n) * square_part(n), Nat.1)
            square_part(n) * square_part(n) = Nat.1
            square_part_pos(n)
            Nat.1 <= square_part(n)
            lte_mul_both(square_part(n), Nat.1, square_part(n))
            square_part(n) * Nat.1 <= square_part(n) * square_part(n)
            square_part(n) * Nat.1 = square_part(n)
            square_part(n) <= Nat.1
            lte_antisymm(square_part(n), Nat.1)
            square_part(n) = Nat.1
            squarefree_of_square_part_one(n)
            is_squarefree(n)
        }
        (is_squarefree(n) implies squarefree_part(n) = n)
        ((squarefree_part(n) = n) implies is_squarefree(n))
        (is_squarefree(n) = (squarefree_part(n) = n))
    }
}
