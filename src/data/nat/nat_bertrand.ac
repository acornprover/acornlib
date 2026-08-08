from nat import Nat, lte_trans, lte_and_lt, lt_and_lte, lt_trans, lte_mul_both,
    add_sub, add_comm, sum_lte, mul_two_left, lt_or_lte, lt_imp_lte_suc,
    pow_pow, add_suc_right, add_suc_left, lte_add_left, lte_ref, lte_antisymm,
    lt_cancel_mul, mul_one_right, mul_comm, add_cancels_right, only_zero_lte_zero,
    divides_lte, lt_suc, lt_add_left, lt_mul_both, zero_or_suc, add_imp_sub,
    mul_cancel_right, suc_sub_one, divides_factorial, mul_cancel_left,
    mul_assoc, lte_suc_suc, sub_one_lt, lte_add_right, add_assoc,
    add_imp_sub_left, divides_trans
from list import partial
from combinatorics import binom, binom_le_binom_suc, binom_suc_le_binom,
    choose_out_of_bounds, factorial_nonzero, choose_add
from number_theory import central_binom, central_binom_eq, central_binom_ne_zero,
    central_binom_legendre, prime_factor_count_upto, prime_factor_count_upto_zero,
    prime_factor_count_upto_step, count_prime_factor, prime_divides_iff_count_ne_zero,
    falling_product, falling_product_suc, falling_product_zero,
    falling_product_eq_binom_mul_factorial, falling_product_prime_count_sum,
    count_prime_factor_falling_product, prime_divides_mul, nat_two_prime,
    falling_product_mul_factorial_complement
from data.nat.nat_range_sum import range_sum, range_sum_le_count_mul
from data.nat.nat_range_sum_bridge import range_sum_eq_partial
from data.nat.nat_binom_row_bound import binom_row, binom_row_sum
from data.nat.nat_central_binom_prime import prime_factor_count_upto_zero_of_lt
from data.nat.nat_prime_central_binom import prime_divides_factorial_imp_lte
from falling_product_prime_factors import prime_divides_falling_product_factor

numerals Nat

/// The middle coefficient of an even row dominates the left half of the row.
///
/// The standard unimodality of a row of binomial coefficients: moving left from the middle the
/// coefficients do not grow. Stated for row `2n` at the middle index `n`, and proved by
/// induction on the distance `n - k` using the left-half monotonicity of a row.
theorem binom_central_max_left(n: Nat, k: Nat) {
    k <= n implies (n + n).binom(k) <= (n + n).binom(n)
} by {
    if k <= n {
        define p(d: Nat) -> Bool {
            forall(j: Nat) {
                j + d = n implies (n + n).binom(j) <= (n + n).binom(n)
            }
        }
        forall(j: Nat) {
            if j + Nat.0 = n {
                (j + Nat.0 = j)
                j = n
                (n + n).binom(j) = (n + n).binom(n)
                (n + n).binom(j) <= (n + n).binom(n)
            }
            (j + Nat.0 = n implies (n + n).binom(j) <= (n + n).binom(n))
        }
        p(Nat.0)
        forall(d: Nat) {
            if p(d) {
                forall(j: Nat) {
                    if j + d.suc = n {
                        add_suc_right(j, d)
                        add_suc_left(j, d)
                        (j + d.suc = j.suc + d)
                        j.suc + d = n
                        (p(d) = forall(j2: Nat) {
                            j2 + d = n implies (n + n).binom(j2) <= (n + n).binom(n)
                        })
                        (j.suc + d = n implies (n + n).binom(j.suc) <= (n + n).binom(n))
                        (n + n).binom(j.suc) <= (n + n).binom(n)
                        j.suc <= j.suc + d
                        lte_trans(j.suc, j.suc + d, n)
                        j.suc <= n
                        lte_mul_both(Nat.2, j.suc, n)
                        Nat.2 * j.suc <= Nat.2 * n
                        mul_two_left(j.suc)
                        (Nat.2 * j.suc = j.suc + j.suc)
                        mul_two_left(n)
                        (Nat.2 * n = n + n)
                        (j.suc + j.suc <= n + n)
                        binom_le_binom_suc(n + n, j)
                        (n + n).binom(j) <= (n + n).binom(j.suc)
                        lte_trans((n + n).binom(j), (n + n).binom(j.suc), (n + n).binom(n))
                        (n + n).binom(j) <= (n + n).binom(n)
                    }
                    (j + d.suc = n implies (n + n).binom(j) <= (n + n).binom(n))
                }
                p(d.suc)
            }
            (p(d) implies p(d.suc))
        }
        p(Nat.0) and forall(d: Nat) {
            p(d) implies p(d.suc)
        }
        Nat.induction(p)
        p(n - k)
        (p(n - k) = forall(j: Nat) {
            j + (n - k) = n implies (n + n).binom(j) <= (n + n).binom(n)
        })
        add_sub(n, k)
        (n - k + k = n)
        add_comm(n - k, k)
        k + (n - k) = n
        (k + (n - k) = n implies (n + n).binom(k) <= (n + n).binom(n))
        (n + n).binom(k) <= (n + n).binom(n)
    }
}

/// The middle coefficient of an even row dominates the right half of the row.
///
/// The mirror image of `binom_central_max_left`, moving right from the middle the coefficients
/// do not grow. Proved by induction on the distance `k - n` using the right-half monotonicity
/// of a row.
theorem binom_central_max_right(n: Nat, k: Nat) {
    n <= k and k <= n + n implies (n + n).binom(k) <= (n + n).binom(n)
} by {
    if n <= k and k <= n + n {
        define q(d: Nat) -> Bool {
            forall(j: Nat) {
                n + d = j implies (n + n).binom(j) <= (n + n).binom(n)
            }
        }
        forall(j: Nat) {
            if n + Nat.0 = j {
                (n + Nat.0 = n)
                j = n
                (n + n).binom(j) = (n + n).binom(n)
                (n + n).binom(j) <= (n + n).binom(n)
            }
            (n + Nat.0 = j implies (n + n).binom(j) <= (n + n).binom(n))
        }
        q(Nat.0)
        forall(d: Nat) {
            if q(d) {
                forall(j: Nat) {
                    if n + d.suc = j {
                        add_suc_right(n, d)
                        (n + d.suc = (n + d).suc)
                        (n + d).suc = j
                        (n + n).binom(j) = (n + n).binom((n + d).suc)
                        n <= n + d
                        n <= n + d
                        sum_lte(n, n, n + d, n + d)
                        n + n <= (n + d) + (n + d)
                        mul_two_left(n + d)
                        (Nat.2 * (n + d) = (n + d) + (n + d))
                        (n + n <= Nat.2 * (n + d))
                        binom_suc_le_binom(n + n, n + d)
                        (n + n).binom((n + d).suc) <= (n + n).binom(n + d)
                        (n + n).binom(j) <= (n + n).binom(n + d)
                        (q(d) = forall(j2: Nat) {
                            n + d = j2 implies (n + n).binom(j2) <= (n + n).binom(n)
                        })
                        (n + d = n + d implies (n + n).binom(n + d) <= (n + n).binom(n))
                        (n + n).binom(n + d) <= (n + n).binom(n)
                        lte_trans((n + n).binom(j), (n + n).binom(n + d), (n + n).binom(n))
                        (n + n).binom(j) <= (n + n).binom(n)
                    }
                    (n + d.suc = j implies (n + n).binom(j) <= (n + n).binom(n))
                }
                q(d.suc)
            }
            (q(d) implies q(d.suc))
        }
        q(Nat.0) and forall(d: Nat) {
            q(d) implies q(d.suc)
        }
        Nat.induction(q)
        q(k - n)
        (q(k - n) = forall(j: Nat) {
            n + (k - n) = j implies (n + n).binom(j) <= (n + n).binom(n)
        })
        add_sub(k, n)
        (k - n + n = k)
        add_comm(k - n, n)
        n + (k - n) = k
        (n + (k - n) = k implies (n + n).binom(k) <= (n + n).binom(n))
        (n + n).binom(k) <= (n + n).binom(n)
    }
}

/// The middle coefficient of an even row dominates the whole row.
///
/// The two halves together: every coefficient of row `2n` is at most the middle one.
theorem binom_le_central(n: Nat, k: Nat) {
    k <= n + n implies (n + n).binom(k) <= (n + n).binom(n)
} by {
    if k <= n + n {
        if k <= n {
            binom_central_max_left(n, k)
            (n + n).binom(k) <= (n + n).binom(n)
        }
        if not (k <= n) {
            not (k <= n)
            lt_or_lte(n, k)
            (n < k or k <= n)
            n < k
            n <= k
            binom_central_max_right(n, k)
            (n + n).binom(k) <= (n + n).binom(n)
        }
        (n + n).binom(k) <= (n + n).binom(n)
    }
}

/// The central binomial coefficient is at least four to the `n` over `2n + 1`.
///
/// The lower half of the two-sided estimate: the row `2n` sums to `4^n`, the middle
/// coefficient is the largest term, and there are `2n + 1` terms.
theorem central_binom_lower_bound(n: Nat) {
    Nat.4.pow(n) <= (n + n + Nat.1) * central_binom(n)
} by {
    binom_row_sum(n + n)
    (partial(binom_row(n + n), (n + n).suc) = Nat.2.pow(n + n))
    mul_two_left(n)
    (n + n = Nat.2 * n)
    (Nat.2.pow(n + n) = Nat.2.pow(Nat.2 * n))
    pow_pow[Nat](Nat.2, Nat.2, n)
    (Nat.2.pow(Nat.2 * n) = Nat.2.pow(Nat.2).pow(n))
    (Nat.2.pow(Nat.2) = Nat.4)
    (Nat.2.pow(n + n) = Nat.4.pow(n))
    (partial(binom_row(n + n), (n + n).suc) = Nat.4.pow(n))
    range_sum_eq_partial(binom_row(n + n), (n + n).suc)
    (range_sum(binom_row(n + n), (n + n).suc) = partial(binom_row(n + n), (n + n).suc))
    (range_sum(binom_row(n + n), (n + n).suc) = Nat.4.pow(n))
    forall(i: Nat) {
        if i <= n + n {
            binom_le_central(n, i)
            (n + n).binom(i) <= (n + n).binom(n)
            central_binom_eq(n)
            (central_binom(n) = (n + n).binom(n))
            (n + n).binom(i) <= central_binom(n)
            (binom_row(n + n, i) = (n + n).binom(i))
            binom_row(n + n)(i) <= central_binom(n)
        }
        if not (i <= n + n) {
            not (i <= n + n)
            lt_or_lte(n + n, i)
            (n + n < i or i <= n + n)
            n + n < i
            choose_out_of_bounds(n + n, i)
            (n + n).binom(i) = Nat.0
            (binom_row(n + n, i) = (n + n).binom(i))
            binom_row(n + n)(i) = Nat.0
            Nat.0 <= central_binom(n)
            binom_row(n + n)(i) <= central_binom(n)
        }
        binom_row(n + n)(i) <= central_binom(n)
    }
    range_sum_le_count_mul(binom_row(n + n), central_binom(n), (n + n).suc)
    (range_sum(binom_row(n + n), (n + n).suc) <= (n + n).suc * central_binom(n))
    ((n + n).suc = n + n + Nat.1)
    (range_sum(binom_row(n + n), (n + n).suc) <= (n + n + Nat.1) * central_binom(n))
    Nat.4.pow(n) <= (n + n + Nat.1) * central_binom(n)
}

/// A prime has no multiple strictly between `p` and `2p`.
///
/// The first gap in the multiples of `p`: the only multiples of a prime `p` below `3p` are
/// `p` and `2p`, so nothing between `p` and `2p` is divisible by `p`.
theorem not_divides_between_p_and_2p(p: Nat, k: Nat) {
    p.is_prime and p < k and k < p + p implies not p.divides(k)
} by {
    if p.is_prime and p < k and k < p + p {
        if p.divides(k) {
            (p.divides(k) = exists(c: Nat) { p * c = k })
            let (c: Nat) satisfy {
                p * c = k
            }
            (p.is_prime = (Nat.1 < p and not p.is_composite))
            Nat.1 < p
            Nat.0 < Nat.1
            lt_trans(Nat.0, Nat.1, p)
            Nat.0 < p
            p != Nat.0
            (p * c < p + p)
            mul_two_left(p)
            (Nat.2 * p = p + p)
            (p * c < Nat.2 * p)
            mul_comm(p, c)
            (c * p < Nat.2 * p)
            mul_two_left(c)
            (Nat.2 * c = c + c)
            (p * Nat.1 < p * c)
            mul_one_right(p)
            (p * Nat.1 = p)
            (p < p * c)
            lt_cancel_mul(p, Nat.1, c)
            Nat.1 < c
            lt_cancel_mul(p, c, Nat.2)
            c < Nat.2
            (c < Nat.2) = (c < Nat.1.suc)
            c <= Nat.1
            lt_and_lte(Nat.1, c, Nat.1)
            Nat.1 < Nat.1
            false
        }
        not p.divides(k)
    }
}

/// A prime has no multiple strictly between `2p` and `3p`.
///
/// The second gap: nothing between `2p` and `3p` is divisible by `p`.
theorem not_divides_between_2p_and_3p(p: Nat, k: Nat) {
    p.is_prime and p + p < k and k < Nat.3 * p implies not p.divides(k)
} by {
    if p.is_prime and p + p < k and k < Nat.3 * p {
        if p.divides(k) {
            (p.divides(k) = exists(c: Nat) { p * c = k })
            let (c: Nat) satisfy {
                p * c = k
            }
            (p.is_prime = (Nat.1 < p and not p.is_composite))
            Nat.1 < p
            Nat.0 < Nat.1
            lt_trans(Nat.0, Nat.1, p)
            Nat.0 < p
            p != Nat.0
            (p * c < Nat.3 * p)
            (p * c < p * Nat.3)
            lt_cancel_mul(p, c, Nat.3)
            c < Nat.3
            (p + p < p * c)
            mul_two_left(p)
            (p * Nat.2 = p + p)
            (p * Nat.2 < p * c)
            lt_cancel_mul(p, Nat.2, c)
            Nat.2 < c
            (c < Nat.3) = (c < Nat.2.suc)
            c <= Nat.2
            lt_and_lte(Nat.2, c, Nat.2)
            Nat.2 < Nat.2
            false
        }
        not p.divides(k)
    }
}

/// Subtracting one more from the subtrahend lowers the difference by one.
///
/// `a - (b + 1)` is one less than `a - b`, so its successor recovers `a - b`, once `b` is
/// strictly below `a`.
theorem suc_of_sub_suc(a: Nat, b: Nat) {
    b < a implies (a - b.suc).suc = a - b
} by {
    if b < a {
        (b < a) = (b <= a and b != a)
        b <= a
        b != a
        (b <= a) = exists(c: Nat) { b + c = a }
        let (d: Nat) satisfy {
            b + d = a
        }
        if d = Nat.0 {
            (b + Nat.0 = b)
            (b + d = b)
            a = b
            b != a
            false
        }
        d != Nat.0
        zero_or_suc(d)
        let (e: Nat) satisfy {
            d = e.suc
        }
        (b + e.suc = a)
        add_suc_right(b, e)
        add_suc_left(b, e)
        (b + e.suc = b.suc + e)
        b.suc + e = a
        suc_sub_one(e)
        (e.suc - Nat.1 = e)
        (d - Nat.1 = e)
        (b.suc + (d - Nat.1) = a)
        add_imp_sub(b.suc, d - Nat.1, a)
        a - b.suc = d - Nat.1
        (d - Nat.1).suc = e.suc
        (d - Nat.1).suc = d
        add_imp_sub(b, d, a)
        a - b = d
        (a - b.suc).suc = a - b
    }
}

/// Subtracting a difference from a difference cancels the common part.
///
/// `(a - b) - (c - b) = a - c` when `b` is at most `c` and `c` is at most `a`.
theorem sub_sub_cancel(a: Nat, b: Nat, c: Nat) {
    b <= c and c <= a implies (a - b) - (c - b) = a - c
} by {
    if b <= c and c <= a {
        (b <= c) = exists(e2: Nat) { b + e2 = c }
        let (e: Nat) satisfy {
            b + e = c
        }
        (c <= a) = exists(f2: Nat) { c + f2 = a }
        let (f: Nat) satisfy {
            c + f = a
        }
        (b + e + f = a)
        add_imp_sub(b, e + f, a)
        a - b = e + f
        add_imp_sub(b, e, c)
        c - b = e
        (e + f = e + f)
        add_imp_sub(e, f, e + f)
        (e + f) - e = f
        add_imp_sub(c, f, a)
        a - c = f
        ((a - b) - (c - b) = (e + f) - e)
        ((e + f) - e = a - c)
        (a - b) - (c - b) = a - c
    }
}

/// Subtraction preserves strict inequality above the subtrahend.
///
/// `b - a < c - a` whenever `a` is at most `b` and `b` is below `c`.
theorem sub_lt_sub(a: Nat, b: Nat, c: Nat) {
    a <= b and b < c implies b - a < c - a
} by {
    if a <= b and b < c {
        (b < c) = (b <= c and b != c)
        b <= c
        b != c
        (a <= b) = exists(e2: Nat) { a + e2 = b }
        let (e: Nat) satisfy {
            a + e = b
        }
        (b <= c) = exists(d2: Nat) { b + d2 = c }
        let (d: Nat) satisfy {
            b + d = c
        }
        if d = Nat.0 {
            (b + Nat.0 = b)
            (b + d = b)
            c = b
            b != c
            false
        }
        d != Nat.0
        (a + e + d = c)
        add_imp_sub(a, e + d, c)
        c - a = e + d
        add_imp_sub(a, e, b)
        b - a = e
        Nat.0 < d
        lt_add_left(e, Nat.0, d)
        (e + Nat.0 < e + d)
        (e + Nat.0 = e)
        e < e + d
        b - a < c - a
    }
}

/// Subtracting a larger subtrahend leaves a smaller difference.
///
/// The standard reversal: `a - b <= a - c` when `c <= b <= a`.
theorem sub_lte_sub(a: Nat, b: Nat, c: Nat) {
    c <= b and b <= a implies a - b <= a - c
} by {
    if c <= b and b <= a {
        (c <= b) = exists(e2: Nat) { c + e2 = b }
        let (e: Nat) satisfy {
            c + e = b
        }
        (b <= a) = exists(f2: Nat) { b + f2 = a }
        let (f: Nat) satisfy {
            b + f = a
        }
        (c + e + f = a)
        add_imp_sub(c, e + f, a)
        a - c = e + f
        add_imp_sub(b, f, a)
        a - b = f
        f <= e + f
        a - b <= a - c
    }
}

/// Subtracting two subtrahends subtracts their sum.
///
/// `(a - b) - c = a - (b + c)` when `b + c` is at most `a`.
theorem sub_sub_assoc(a: Nat, b: Nat, c: Nat) {
    b + c <= a implies (a - b) - c = a - (b + c)
} by {
    if b + c <= a {
        (b + c <= a) = exists(d2: Nat) { b + c + d2 = a }
        let (d: Nat) satisfy {
            b + c + d = a
        }
        (b + (c + d) = a)
        add_imp_sub(b, c + d, a)
        a - b = c + d
        (c + d = c + d)
        add_imp_sub(c, d, c + d)
        (c + d) - c = d
        (a - b) - c = d
        add_imp_sub(b + c, d, a)
        a - (b + c) = d
        (a - b) - c = a - (b + c)
    }
}

/// Moving an addend across a subtraction.
///
/// `(a - b) + c = (a + c) - b` when `b` is at most `a`.
theorem add_sub_comm(a: Nat, b: Nat, c: Nat) {
    b <= a implies (a - b) + c = (a + c) - b
} by {
    if b <= a {
        (b <= a) = exists(e2: Nat) { b + e2 = a }
        let (e: Nat) satisfy {
            b + e = a
        }
        (a - b = e)
        ((a - b) + c = e + c)
        (a + c = b + e + c)
        add_imp_sub(b, e + c, a + c)
        (a + c) - b = e + c
        (a - b) + c = (a + c) - b
    }
}

/// Subtracting successors cancels the successors.
///
/// `(b + 1) - (a + 1) = b - a` when `a` is below `b`.
theorem suc_sub_suc_eq(a: Nat, b: Nat) {
    a < b implies b.suc - a.suc = b - a
} by {
    if a < b {
        (a < b) = (a <= b and a != b)
        a <= b
        (a <= b) = exists(d2: Nat) { a + d2 = b }
        let (d: Nat) satisfy {
            a + d = b
        }
        add_suc_left(a, d)
        (a.suc + d = (a + d).suc)
        (a.suc + d = b.suc)
        add_imp_sub(a.suc, d, b.suc)
        b.suc - a.suc = d
        add_imp_sub(a, d, b)
        b - a = d
        b.suc - a.suc = b - a
    }
}

/// A falling product splits at any position.
///
/// The factors `n, n - 1, ..., n - k` split into those above `n - a` and those from `n - a - 1`
/// down. This is the tool the valuation analysis uses to isolate one factor of a falling
/// product.
theorem falling_product_split(n: Nat, k: Nat, a: Nat) {
    a < k and k < n implies falling_product(n, k) = falling_product(n, a) * falling_product(n - a.suc, k - a.suc)
} by {
    if a < k and k < n {
        falling_product_mul_factorial_complement(n, k)
        (falling_product(n, k) * (n - k.suc).factorial = n.factorial)
        lt_trans(a, k, n)
        a < n
        falling_product_mul_factorial_complement(n, a)
        (falling_product(n, a) * (n - a.suc).factorial = n.factorial)
        a <= k
        sub_lt_sub(a.suc, k, n)
        (k - a.suc < n - a.suc)
        falling_product_mul_factorial_complement(n - a.suc, k - a.suc)
        (falling_product(n - a.suc, k - a.suc)
            * (n - a.suc - (k - a.suc).suc).factorial = (n - a.suc).factorial)
        suc_of_sub_suc(k, a)
        ((k - a.suc).suc = k - a)
        (a < k) = (a <= k and a != k)
        a <= k
        (a <= k) = exists(d2: Nat) { a + d2 = k }
        let (d: Nat) satisfy {
            a + d = k
        }
        add_suc_left(a, d)
        (a.suc + d = (a + d).suc)
        (a.suc + d = k.suc)
        add_imp_sub(a.suc, d, k.suc)
        k.suc - a.suc = d
        add_imp_sub(a, d, k)
        k - a = d
        (k.suc - a.suc = k - a)
        ((k - a.suc).suc = k.suc - a.suc)
        a <= k
        lt_imp_lte_suc(k, n)
        (k.suc <= n)
        (a.suc <= k.suc)
        (a.suc <= k.suc and k.suc <= n)
        sub_sub_cancel(n, a.suc, k.suc)
        (a.suc <= k.suc and k.suc <= n implies (n - a.suc) - (k.suc - a.suc) = n - k.suc)
        ((n - a.suc) - (k.suc - a.suc) = n - k.suc)
        ((n - a.suc) - (k - a.suc).suc = n - k.suc)
        ((falling_product(n - a.suc, k - a.suc)
            * (n - k.suc).factorial) = (n - a.suc).factorial)
        ((falling_product(n, a) * falling_product(n - a.suc, k - a.suc)
            * (n - k.suc).factorial) = falling_product(n, a) * (n - a.suc).factorial)
        (falling_product(n, a) * falling_product(n - a.suc, k - a.suc)
            * (n - k.suc).factorial = n.factorial)
        factorial_nonzero(n - k.suc)
        (n - k.suc).factorial != Nat.0
        mul_cancel_right((n - k.suc).factorial, falling_product(n, k),
            falling_product(n, a) * falling_product(n - a.suc, k - a.suc))
        falling_product(n, k) = falling_product(n, a) * falling_product(n - a.suc, k - a.suc)
    }
}

/// A prime strictly between `2n/3` and `n` does not divide the central binomial coefficient.
///
/// The heart of the elementary proof of Bertrand's postulate. With `p` in `(2n/3, n)`, the
/// falling product `(2n)(2n - 1)...(n + 1)` is `p` times a factor-free-of-`p` remainder
/// together with `2p`: exactly one factor, `2p`, is divisible by `p`, and it is divisible to
/// the first power only. If `p` divided the central binomial coefficient, the falling product
/// would carry `p` at least twice (once from the coefficient, once from `n!`), which is
/// impossible.
theorem prime_lt_n_two_thirds_not_divides(p: Nat, n: Nat) {
    p.is_prime and Nat.2 < p and p < n and Nat.2 * n < Nat.3 * p
        implies not p.divides(central_binom(n))
} by {
    if p.is_prime and Nat.2 < p and p < n and Nat.2 * n < Nat.3 * p {
        p <= n
        lt_trans(Nat.2, p, n)
        Nat.2 < n
        Nat.1 < Nat.2
        lt_trans(Nat.1, Nat.2, n)
        Nat.1 < n
        Nat.1 <= n
        n != Nat.0
        Nat.0 < n
        sub_one_lt(n)
        (n - Nat.1 < n)
        lt_add_left(n, Nat.0, n)
        (n + Nat.0 < n + n)
        (n + Nat.0 = n)
        (n < n + n)
        lt_trans(n - Nat.1, n, n + n)
        (n - Nat.1 < n + n)
        falling_product_eq_binom_mul_factorial(n + n, n - Nat.1)
        (falling_product(n + n, n - Nat.1)
            = (n + n).binom((n - Nat.1).suc) * (n - Nat.1).suc.factorial)
        add_sub(n, Nat.1)
        (n - Nat.1 + Nat.1 = n)
        ((n - Nat.1).suc = n)
        (falling_product(n + n, n - Nat.1) = (n + n).binom(n) * n.factorial)
        central_binom_eq(n)
        (central_binom(n) = (n + n).binom(n))
        (falling_product(n + n, n - Nat.1) = central_binom(n) * n.factorial)
        if p.divides(central_binom(n)) {
            (p.divides(central_binom(n)) = exists(c1: Nat) { p * c1 = central_binom(n) })
            let (c1: Nat) satisfy {
                p * c1 = central_binom(n)
            }
            p != Nat.0
            divides_factorial(p, n)
            (p != Nat.0 and p <= n implies p.divides(n.factorial))
            p.divides(n.factorial)
            (p.divides(n.factorial) = exists(c2: Nat) { p * c2 = n.factorial })
            let (c2: Nat) satisfy {
                p * c2 = n.factorial
            }
            (falling_product(n + n, n - Nat.1) = (p * c1) * (p * c2))
            mul_assoc(p, c1, p * c2)
            ((p * c1) * (p * c2) = p * (c1 * (p * c2)))
            mul_comm(p, c2)
            (c1 * (p * c2) = c1 * (c2 * p))
            mul_assoc(c1, c2, p)
            (c1 * (c2 * p) = (c1 * c2) * p)
            mul_comm(c1 * c2, p)
            ((c1 * c2) * p = p * (c1 * c2))
            ((p * c1) * (p * c2) = p * (p * (c1 * c2)))
            (falling_product(n + n, n - Nat.1) = p * (p * (c1 * c2)))
            // n < 2p from 2n < 3p
            p != Nat.0
            (Nat.3 < Nat.4)
            lt_mul_both(p, Nat.3, Nat.4)
            (p * Nat.3 < p * Nat.4)
            mul_comm(p, Nat.3)
            (Nat.3 * p = p * Nat.3)
            mul_comm(p, Nat.4)
            (Nat.4 * p = p * Nat.4)
            (Nat.3 * p < Nat.4 * p)
            lt_trans(Nat.2 * n, Nat.3 * p, Nat.4 * p)
            (Nat.2 * n < Nat.4 * p)
            (Nat.4 * p = Nat.2 * (Nat.2 * p))
            (Nat.2 * n < Nat.2 * (Nat.2 * p))
            Nat.2 != Nat.0
            lt_cancel_mul(Nat.2, n, Nat.2 * p)
            (n < Nat.2 * p)
            mul_two_left(p)
            (Nat.2 * p = p + p)
            n < p + p
            // 2p - 1 < 2n from p < n
            Nat.2 != Nat.0
            lt_mul_both(Nat.2, p, n)
            (Nat.2 * p < Nat.2 * n)
            mul_two_left(p)
            (Nat.2 * p = p + p)
            mul_two_left(n)
            (Nat.2 * n = n + n)
            (p + p < n + n)
            p != Nat.0
            zero_or_suc(p)
            let (m1: Nat) satisfy {
                p = m1.suc
            }
            (Nat.0.suc = Nat.1)
            lte_suc_suc(Nat.0, m1)
            (Nat.1 <= m1.suc)
            (Nat.1 <= p)
            Nat.0 < Nat.1
            lte_and_lt(Nat.0, Nat.1, p)
            Nat.0 < p
            lt_add_left(p, Nat.0, p)
            (p + Nat.0 < p + p)
            (p + Nat.0 = p)
            (p < p + p)
            (Nat.0 <= p)
            lte_and_lt(Nat.0, p, p + p)
            Nat.0 < p + p
            sub_one_lt(p + p)
            (p + p - Nat.1 < p + p)
            lt_trans(p + p - Nat.1, p + p, n + n)
            (p + p - Nat.1 < n + n)
            // 2n - 2p < p, hence 2n - 2p < n
            (Nat.2 * p <= Nat.2 * n)
            sub_lt_sub(Nat.2 * p, Nat.2 * n, Nat.3 * p)
            (Nat.2 * p <= Nat.2 * n and Nat.2 * n < Nat.3 * p
                implies Nat.2 * n - Nat.2 * p < Nat.3 * p - Nat.2 * p)
            (Nat.2 * n - Nat.2 * p < Nat.3 * p - Nat.2 * p)
            (Nat.3 * p = Nat.2 * p + p)
            (Nat.3 * p - Nat.2 * p = p)
            (Nat.2 * n - Nat.2 * p < p)
            lt_trans(Nat.2 * n - Nat.2 * p, p, n)
            (Nat.2 * n - Nat.2 * p < n)
            // 1 <= 2n - 2p
            (Nat.2 * p < Nat.2 * n)
            lt_imp_lte_suc(Nat.2 * p, Nat.2 * n)
            ((Nat.2 * p).suc <= Nat.2 * n)
            ((Nat.2 * p).suc = Nat.2 * p + Nat.1)
            (Nat.2 * p + Nat.1 <= Nat.2 * n)
            (Nat.2 * p + Nat.1 <= Nat.2 * n) = exists(d2: Nat) { Nat.2 * p + Nat.1 + d2 = Nat.2 * n }
            let (d: Nat) satisfy {
                Nat.2 * p + Nat.1 + d = Nat.2 * n
            }
            (Nat.2 * p + (Nat.1 + d) = Nat.2 * n)
            add_imp_sub(Nat.2 * p, Nat.1 + d, Nat.2 * n)
            (Nat.2 * n - Nat.2 * p = Nat.1 + d)
            Nat.1 <= Nat.1 + d
            (Nat.1 <= Nat.2 * n - Nat.2 * p)
            sub_lt_sub(Nat.1, Nat.2 * n - Nat.2 * p, n)
            (Nat.1 <= Nat.2 * n - Nat.2 * p and Nat.2 * n - Nat.2 * p < n
                implies (Nat.2 * n - Nat.2 * p) - Nat.1 < n - Nat.1)
            ((Nat.2 * n - Nat.2 * p) - Nat.1 < n - Nat.1)
            mul_two_left(n)
            (Nat.2 * n = n + n)
            mul_two_left(p)
            (Nat.2 * p = p + p)
            (p + p <= n + n)
            sub_sub_assoc(n + n, p, p)
            (p + p <= n + n implies (n + n - p) - p = (n + n) - (p + p))
            ((n + n - p) - p = (n + n) - (p + p))
            (n + n - p - p = (n + n) - (p + p))
            (Nat.2 * n - Nat.2 * p = (n + n) - (p + p))
            (Nat.2 * n - Nat.2 * p = n + n - p - p)
            ((Nat.2 * n - Nat.2 * p) - Nat.1 = (n + n - p - p) - Nat.1)
            ((n + n - p - p) - Nat.1 = n + n - p - p - Nat.1)
            ((Nat.2 * n - Nat.2 * p) - Nat.1 = n + n - p - p - Nat.1)
            (n + n - p - p - Nat.1 < n - Nat.1)
            // split fp at position 2n - 2p - 1
            ((n + n - p - p - Nat.1).suc = n + n - p - p)
            falling_product_split(n + n, n - Nat.1, n + n - p - p - Nat.1)
            (n + n - p - p - Nat.1 < n - Nat.1 and n - Nat.1 < n + n
                implies falling_product(n + n, n - Nat.1)
                    = falling_product(n + n, n + n - p - p - Nat.1)
                        * falling_product(n + n - (n + n - p - p - Nat.1).suc,
                            (n - Nat.1) - (n + n - p - p - Nat.1).suc))
            (falling_product(n + n, n - Nat.1)
                = falling_product(n + n, n + n - p - p - Nat.1)
                    * falling_product(n + n - (n + n - p - p - Nat.1).suc,
                        (n - Nat.1) - (n + n - p - p - Nat.1).suc))
            (n + n - (n + n - p - p - Nat.1).suc = p + p)
            (falling_product(n + n, n - Nat.1)
                = falling_product(n + n, n + n - p - p - Nat.1)
                    * falling_product(p + p, (n - Nat.1) - (n + n - p - p - Nat.1).suc))
            // suffix length: (n-1) - (2n-2p) = 2p - n - 1
            (p + p <= n + n)
            add_sub(n + n, p + p)
            (n + n - p - p + (p + p) = n + n)
            (n + n - p - p) + (p + p) = n + n
            add_imp_sub(n + n - p - p, p + p, n + n)
            (n + n) - (n + n - p - p) = p + p
            ((n + n - p - p - Nat.1).suc = n + n - p - p)
            ((n + n) - (n + n - p - p - Nat.1).suc = p + p)
            (n + n - (n + n - p - p - Nat.1).suc = p + p)
            (falling_product(n + n, n - Nat.1)
                = falling_product(n + n, n + n - p - p - Nat.1)
                    * falling_product(p + p, (n - Nat.1) - (n + n - p - p - Nat.1).suc))
            // (n-1) - (2n-2p) = 2p - n - 1
            sub_sub_assoc(p + p, n, Nat.1)
            (n + Nat.1 <= p + p implies (p + p - n) - Nat.1 = (p + p) - (n + Nat.1))
            (n + Nat.1 <= p + p)
            (p + p - n - Nat.1 = (p + p - n) - Nat.1)
            // (2p - n) + (2n - 2p) = n
            (n <= p + p)
            add_sub_comm(p + p, n, n + n - p - p)
            (n <= p + p implies (p + p - n) + (n + n - p - p)
                = (p + p + (n + n - p - p)) - n)
            ((p + p - n) + (n + n - p - p)
                = (p + p + (n + n - p - p)) - n)
            (p + p + (n + n - p - p) = n + n)
            ((p + p - n) + (n + n - p - p) = (n + n) - n)
            ((n + n) - n = n)
            ((p + p - n) + (n + n - p - p) = n)
            // 1 <= 2p - n from n + 1 <= 2p
            (n < p + p)
            lt_imp_lte_suc(n, p + p)
            (n.suc <= p + p)
            (n.suc = n + Nat.1)
            (n + Nat.1 <= p + p)
            (n + Nat.1 <= p + p) = exists(g2: Nat) { n + Nat.1 + g2 = p + p }
            let (g: Nat) satisfy {
                n + Nat.1 + g = p + p
            }
            add_imp_sub(n + Nat.1, g, p + p)
            (p + p) - (n + Nat.1) = g
            (n + Nat.1 <= p + p)
            add_sub_comm(p + p, n + Nat.1, Nat.1)
            (n + Nat.1 <= p + p implies ((p + p) - (n + Nat.1)) + Nat.1
                = ((p + p) + Nat.1) - (n + Nat.1))
            (((p + p) - (n + Nat.1)) + Nat.1 = ((p + p) + Nat.1) - (n + Nat.1))
            (g + Nat.1 = ((p + p) + Nat.1) - (n + Nat.1))
            suc_sub_suc_eq(n, p + p)
            (n < p + p implies (p + p).suc - n.suc = (p + p) - n)
            ((p + p).suc - n.suc = (p + p) - n)
            ((p + p).suc = (p + p) + Nat.1)
            (n.suc = n + Nat.1)
            (((p + p) + Nat.1) - (n + Nat.1) = (p + p) - n)
            (g + Nat.1 = (p + p) - n)
            Nat.1 <= g + Nat.1
            (Nat.1 <= p + p - n)
            add_sub_comm(p + p - n, Nat.1, n + n - p - p)
            (Nat.1 <= p + p - n implies (p + p - n - Nat.1) + (n + n - p - p)
                = ((p + p - n) + (n + n - p - p)) - Nat.1)
            ((p + p - n - Nat.1) + (n + n - p - p)
                = ((p + p - n) + (n + n - p - p)) - Nat.1)
            ((p + p - n - Nat.1) + (n + n - p - p) = n - Nat.1)
            add_imp_sub(n + n - p - p, p + p - n - Nat.1, n - Nat.1)
            ((n - Nat.1) - (n + n - p - p) = p + p - n - Nat.1)
            ((n - Nat.1) - (n + n - p - p - Nat.1).suc = p + p - n - Nat.1)
            (falling_product(n + n, n - Nat.1)
                = falling_product(n + n, n + n - p - p - Nat.1)
                    * falling_product(p + p, p + p - n - Nat.1))
            // 0 < 2p - n - 1 : suppose 2p <= n + 1
            if p + p <= n + Nat.1 {
                (p + p <= n + Nat.1) = exists(e2: Nat) { p + p + e2 = n + Nat.1 }
                let (e: Nat) satisfy {
                    p + p + e = n + Nat.1
                }
                (p + p = Nat.2 * p)
                (Nat.2 * p <= n + Nat.1)
                lte_mul_both(Nat.2, Nat.2 * p, n + Nat.1)
                (Nat.2 * (Nat.2 * p) <= Nat.2 * (n + Nat.1))
                (Nat.2 * (Nat.2 * p) = Nat.4 * p)
                (Nat.2 * (n + Nat.1) = Nat.2 * n + Nat.2)
                (Nat.4 * p <= Nat.2 * n + Nat.2)
                lt_add_left(Nat.2, Nat.2 * n, Nat.3 * p)
                (Nat.2 + Nat.2 * n < Nat.2 + Nat.3 * p)
                (Nat.2 + Nat.2 * n = Nat.2 * n + Nat.2)
                (Nat.2 * n + Nat.2 < Nat.2 + Nat.3 * p)
                (Nat.2 + Nat.3 * p = Nat.3 * p + Nat.2)
                (Nat.2 * n + Nat.2 < Nat.3 * p + Nat.2)
                lte_and_lt(Nat.4 * p, Nat.2 * n + Nat.2, Nat.3 * p + Nat.2)
                (Nat.4 * p < Nat.3 * p + Nat.2)
                (Nat.3 * p <= Nat.4 * p)
                sub_lt_sub(Nat.3 * p, Nat.4 * p, Nat.3 * p + Nat.2)
                (Nat.3 * p <= Nat.4 * p and Nat.4 * p < Nat.3 * p + Nat.2
                    implies Nat.4 * p - Nat.3 * p < (Nat.3 * p + Nat.2) - Nat.3 * p)
                (Nat.4 * p - Nat.3 * p < (Nat.3 * p + Nat.2) - Nat.3 * p)
                (Nat.4 * p - Nat.3 * p = p)
                ((Nat.3 * p + Nat.2) - Nat.3 * p = Nat.2)
                (p < Nat.2)
                (p <= Nat.2)
                lte_and_lt(p, Nat.2, p)
                p < p
                false
            }
            not (p + p <= n + Nat.1)
            n + Nat.1 < p + p
            (n + Nat.1 < p + p) = (n + Nat.1 <= p + p and n + Nat.1 != p + p)
            n + Nat.1 <= p + p
            (n + Nat.1 <= p + p) = exists(c2b: Nat) { n + Nat.1 + c2b = p + p }
            let (c: Nat) satisfy {
                n + Nat.1 + c = p + p
            }
            c != Nat.0
            zero_or_suc(c)
            let (m: Nat) satisfy {
                c = m.suc
            }
            (Nat.0.suc = Nat.1)
            lte_suc_suc(Nat.0, m)
            (Nat.1 <= m.suc)
            (Nat.1 <= c)
            Nat.0 < Nat.1
            lte_and_lt(Nat.0, Nat.1, c)
            Nat.0 < c
            add_imp_sub(n + Nat.1, c, p + p)
            (p + p) - (n + Nat.1) = c
            (n + Nat.1 <= p + p)
            sub_sub_assoc(p + p, n, Nat.1)
            ((p + p) - n - Nat.1 = (p + p) - (n + Nat.1))
            (p + p - n - Nat.1 = c)
            Nat.0 < p + p - n - Nat.1
            // split the suffix at 0
            (Nat.1 <= n)
            (n <= p + p)
            sub_lte_sub(p + p, n, Nat.1)
            (Nat.1 <= n and n <= p + p implies p + p - n <= p + p - Nat.1)
            (p + p - n <= p + p - Nat.1)
            (Nat.0 < p + p - n)
            sub_one_lt(p + p - n)
            (p + p - n - Nat.1 < p + p - n)
            (p + p - n - Nat.1 <= p + p - n)
            lte_trans(p + p - n - Nat.1, p + p - n, p + p - Nat.1)
            (p + p - n - Nat.1 <= p + p - Nat.1)
            (p + p - Nat.1 < p + p)
            lte_and_lt(p + p - n - Nat.1, p + p - Nat.1, p + p)
            (p + p - n - Nat.1 < p + p)
            falling_product_split(p + p, p + p - n - Nat.1, Nat.0)
            (Nat.0 < p + p - n - Nat.1 and p + p - n - Nat.1 < p + p
                implies falling_product(p + p, p + p - n - Nat.1)
                    = falling_product(p + p, Nat.0)
                        * falling_product(p + p - Nat.1, p + p - n - Nat.1 - Nat.1))
            (falling_product(p + p, p + p - n - Nat.1)
                = falling_product(p + p, Nat.0)
                    * falling_product(p + p - Nat.1, p + p - n - Nat.1 - Nat.1))
            falling_product_zero(p + p)
            (falling_product(p + p, Nat.0) = p + p)
            (Nat.0 < p + p - n - Nat.1)
            (Nat.1 <= p + p - n - Nat.1)
            lte_add_right(Nat.1, Nat.1, p + p - n - Nat.1)
            (Nat.1 + Nat.1 <= p + p - n - Nat.1 + Nat.1)
            ((p + p - n - Nat.1) + Nat.1 = p + p - n)
            (Nat.1 + Nat.1 <= p + p - n)
            (p + p - n - Nat.1 - Nat.1 = (p + p - n - Nat.1) - Nat.1)
            sub_sub_assoc(p + p - n, Nat.1, Nat.1)
            (Nat.1 + Nat.1 <= p + p - n implies (p + p - n - Nat.1) - Nat.1
                = (p + p - n) - (Nat.1 + Nat.1))
            ((p + p - n - Nat.1) - Nat.1 = (p + p - n) - (Nat.1 + Nat.1))
            (Nat.1 + Nat.1 = Nat.2)
            ((p + p - n - Nat.1) - Nat.1 = (p + p - n) - Nat.2)
            ((p + p - n) - Nat.2 = p + p - n - Nat.2)
            (p + p - n - Nat.1 - Nat.1 = p + p - n - Nat.2)
            (falling_product(p + p, p + p - n - Nat.1)
                = (p + p) * falling_product(p + p - Nat.1, p + p - n - Nat.2))
            (falling_product(n + n, n - Nat.1)
                = falling_product(n + n, n + n - p - p - Nat.1) * (p + p)
                    * falling_product(p + p - Nat.1, p + p - n - Nat.2))
            mul_two_left(p)
            (Nat.2 * p = p + p)
            (p + p = Nat.2 * p)
            mul_comm(falling_product(n + n, n + n - p - p - Nat.1), p + p)
            mul_assoc(p + p, falling_product(n + n, n + n - p - p - Nat.1),
                falling_product(p + p - Nat.1, p + p - n - Nat.2))
            ((falling_product(n + n, n + n - p - p - Nat.1) * (p + p))
                    * falling_product(p + p - Nat.1, p + p - n - Nat.2)
                = (p + p) * (falling_product(n + n, n + n - p - p - Nat.1)
                    * falling_product(p + p - Nat.1, p + p - n - Nat.2)))
            ((falling_product(n + n, n + n - p - p - Nat.1) * (p + p)
                    * falling_product(p + p - Nat.1, p + p - n - Nat.2))
                = (p + p) * (falling_product(n + n, n + n - p - p - Nat.1)
                    * falling_product(p + p - Nat.1, p + p - n - Nat.2)))
            ((p + p) * (falling_product(n + n, n + n - p - p - Nat.1)
                    * falling_product(p + p - Nat.1, p + p - n - Nat.2))
                = (Nat.2 * p) * (falling_product(n + n, n + n - p - p - Nat.1)
                    * falling_product(p + p - Nat.1, p + p - n - Nat.2)))
            mul_assoc(Nat.2, p,
                falling_product(n + n, n + n - p - p - Nat.1)
                    * falling_product(p + p - Nat.1, p + p - n - Nat.2))
            ((Nat.2 * p) * (falling_product(n + n, n + n - p - p - Nat.1)
                    * falling_product(p + p - Nat.1, p + p - n - Nat.2))
                = Nat.2 * (p * (falling_product(n + n, n + n - p - p - Nat.1)
                    * falling_product(p + p - Nat.1, p + p - n - Nat.2))))
            mul_comm(p, Nat.2)
            (p * Nat.2 = Nat.2 * p)
            mul_assoc(p, Nat.2,
                falling_product(n + n, n + n - p - p - Nat.1)
                    * falling_product(p + p - Nat.1, p + p - n - Nat.2))
            (p * (Nat.2 * (falling_product(n + n, n + n - p - p - Nat.1)
                    * falling_product(p + p - Nat.1, p + p - n - Nat.2)))
                = (p * Nat.2) * (falling_product(n + n, n + n - p - p - Nat.1)
                    * falling_product(p + p - Nat.1, p + p - n - Nat.2)))
            ((p * Nat.2) * (falling_product(n + n, n + n - p - p - Nat.1)
                    * falling_product(p + p - Nat.1, p + p - n - Nat.2))
                = Nat.2 * (p * (falling_product(n + n, n + n - p - p - Nat.1)
                    * falling_product(p + p - Nat.1, p + p - n - Nat.2))))
            (falling_product(n + n, n + n - p - p - Nat.1) * (p + p)
                    * falling_product(p + p - Nat.1, p + p - n - Nat.2)
                = p * (Nat.2 * (falling_product(n + n, n + n - p - p - Nat.1)
                    * falling_product(p + p - Nat.1, p + p - n - Nat.2))))
            (falling_product(n + n, n - Nat.1)
                = p * (Nat.2 * (falling_product(n + n, n + n - p - p - Nat.1)
                    * falling_product(p + p - Nat.1, p + p - n - Nat.2))))
            (p * (p * (c1 * c2))
                = p * (Nat.2 * (falling_product(n + n, n + n - p - p - Nat.1)
                    * falling_product(p + p - Nat.1, p + p - n - Nat.2))))
            mul_cancel_left(p, p * (c1 * c2),
                Nat.2 * (falling_product(n + n, n + n - p - p - Nat.1)
                    * falling_product(p + p - Nat.1, p + p - n - Nat.2)))
            (p * (c1 * c2)
                = Nat.2 * (falling_product(n + n, n + n - p - p - Nat.1)
                    * falling_product(p + p - Nat.1, p + p - n - Nat.2)))
            (p.divides(Nat.2 * (falling_product(n + n, n + n - p - p - Nat.1)
                * falling_product(p + p - Nat.1, p + p - n - Nat.2))))
            prime_divides_mul(p, Nat.2,
                falling_product(n + n, n + n - p - p - Nat.1)
                    * falling_product(p + p - Nat.1, p + p - n - Nat.2))
            (p.divides(Nat.2) or p.divides(falling_product(n + n, n + n - p - p - Nat.1)
                * falling_product(p + p - Nat.1, p + p - n - Nat.2)))
            if p.divides(Nat.2) {
                divides_lte(p, Nat.2)
                (Nat.2 = Nat.0 or p <= Nat.2)
                Nat.2 != Nat.0
                p <= Nat.2
                lte_and_lt(p, Nat.2, p)
                p < p
                false
            }
            not p.divides(Nat.2)
            p.divides(falling_product(n + n, n + n - p - p - Nat.1)
                * falling_product(p + p - Nat.1, p + p - n - Nat.2))
            prime_divides_mul(p, falling_product(n + n, n + n - p - p - Nat.1),
                falling_product(p + p - Nat.1, p + p - n - Nat.2))
            (p.divides(falling_product(n + n, n + n - p - p - Nat.1))
                or p.divides(falling_product(p + p - Nat.1, p + p - n - Nat.2)))
            if p.divides(falling_product(n + n, n + n - p - p - Nat.1)) {
                prime_divides_falling_product_factor(n + n, n + n - p - p - Nat.1, p)
                exists(i: Nat) {
                    i <= n + n - p - p - Nat.1 and p.divides(n + n - i)
                }
                let (i: Nat) satisfy {
                    i <= n + n - p - p - Nat.1 and p.divides(n + n - i)
                }
                (i <= n + n - p - p - Nat.1)
                (Nat.1 <= n + n - p - p)
                Nat.0 < n + n - p - p
                sub_one_lt(n + n - p - p)
                (n + n - p - p - Nat.1 < n + n - p - p)
                (n + n - p - p - Nat.1 <= n + n - p - p)
                (Nat.0 <= p + p)
                (p + p <= n + n)
                sub_lte_sub(n + n, p + p, Nat.0)
                (Nat.0 <= p + p and p + p <= n + n implies (n + n) - (p + p) <= (n + n) - Nat.0)
                ((n + n) - (p + p) <= (n + n) - Nat.0)
                ((n + n) - Nat.0 = n + n)
                ((n + n) - (p + p) <= n + n)
                sub_sub_assoc(n + n, p, p)
                (p + p <= n + n implies (n + n - p) - p = (n + n) - (p + p))
                ((n + n - p) - p = (n + n) - (p + p))
                (n + n - p - p = (n + n) - (p + p))
                (n + n - p - p <= n + n)
                lte_trans(n + n - p - p - Nat.1, n + n - p - p, n + n)
                (n + n - p - p - Nat.1 <= n + n)
                (i <= n + n - p - p - Nat.1 and n + n - p - p - Nat.1 <= n + n)
                sub_lte_sub(n + n, i, n + n - p - p - Nat.1)
                (i <= n + n - p - p - Nat.1 and n + n - p - p - Nat.1 <= n + n
                    implies (n + n) - (n + n - p - p - Nat.1) <= (n + n) - i)
                ((n + n) - (n + n - p - p - Nat.1) <= (n + n) - i)
                (Nat.1 <= n + n - p - p)
                add_sub(n + n - p - p, Nat.1)
                (Nat.1 <= n + n - p - p implies n + n - p - p - Nat.1 + Nat.1 = n + n - p - p)
                (n + n - p - p - Nat.1) + Nat.1 = n + n - p - p
                (((n + n - p - p - Nat.1) + Nat.1) + (p + p) = n + n)
                ((n + n - p - p - Nat.1) + (p + p + Nat.1) = n + n)
                add_imp_sub(n + n - p - p - Nat.1, p + p + Nat.1, n + n)
                ((n + n) - (n + n - p - p - Nat.1) = p + p + Nat.1)
                (p + p + Nat.1 <= (n + n) - i)
                (p + p < p + p + Nat.1)
                (p + p + Nat.1 = (p + p).suc)
                (p + p < (p + p).suc)
                lt_and_lte(p + p, p + p + Nat.1, (n + n) - i)
                (p + p < (n + n) - i)
                (i <= n + n - p - p - Nat.1)
                (n + n - p - p - Nat.1 <= n + n)
                lte_trans(i, n + n - p - p - Nat.1, n + n)
                (i <= n + n)
                sub_lte_sub(n + n, i, Nat.0)
                ((n + n) - i <= (n + n) - Nat.0)
                ((n + n) - Nat.0 = n + n)
                (n + n - i <= n + n)
                (n + n < Nat.3 * p)
                lte_and_lt(n + n - i, n + n, Nat.3 * p)
                (n + n - i < Nat.3 * p)
                not_divides_between_2p_and_3p(p, n + n - i)
                not p.divides(n + n - i)
                false
            }
            if p.divides(falling_product(p + p - Nat.1, p + p - n - Nat.2)) {
                prime_divides_falling_product_factor(p + p - Nat.1, p + p - n - Nat.2, p)
                exists(j: Nat) {
                    j <= p + p - n - Nat.2 and p.divides(p + p - Nat.1 - j)
                }
                let (j: Nat) satisfy {
                    j <= p + p - n - Nat.2 and p.divides(p + p - Nat.1 - j)
                }
                (j <= p + p - n - Nat.2)
                (Nat.0 < p + p - n - Nat.1)
                (p + p - n - Nat.1 > Nat.0)
                sub_one_lt(p + p - n - Nat.1)
                (p + p - n - Nat.1 - Nat.1 < p + p - n - Nat.1)
                (p + p - n - Nat.2 < p + p - n - Nat.1)
                (p + p - n - Nat.2 <= p + p - n - Nat.1)
                (Nat.1 <= n)
                (n <= p + p)
                sub_lte_sub(p + p, n, Nat.1)
                (Nat.1 <= n and n <= p + p implies p + p - n <= p + p - Nat.1)
                (p + p - n <= p + p - Nat.1)
                (p + p - n - Nat.1 <= p + p - n)
                lte_trans(p + p - n - Nat.1, p + p - n, p + p - Nat.1)
                (p + p - n - Nat.1 <= p + p - Nat.1)
                lte_trans(p + p - n - Nat.2, p + p - n - Nat.1, p + p - Nat.1)
                (p + p - n - Nat.2 <= p + p - Nat.1)
                (j <= p + p - n - Nat.2 and p + p - n - Nat.2 <= p + p - Nat.1)
                sub_lte_sub(p + p - Nat.1, j, p + p - n - Nat.2)
                ((p + p - Nat.1) - (p + p - n - Nat.2) <= (p + p - Nat.1) - j)
                // (2p-1) - (2p-n-2) = n+1
                (p + p - n - Nat.2 = (p + p - n) - Nat.2)
                (p + p - n - Nat.1 - Nat.1 = p + p - n - Nat.2)
                (n + Nat.2 <= p + p)
                sub_sub_assoc(p + p, n, Nat.2)
                (n + Nat.2 <= p + p implies (p + p - n) - Nat.2 = (p + p) - (n + Nat.2))
                ((p + p - n) - Nat.2 = (p + p) - (n + Nat.2))
                (p + p - n - Nat.2 = (p + p) - (n + Nat.2))
                add_sub_comm(p + p, n + Nat.2, n + Nat.1)
                (n + Nat.2 <= p + p implies ((p + p) - (n + Nat.2)) + (n + Nat.1)
                    = ((p + p) + (n + Nat.1)) - (n + Nat.2))
                (((p + p) - (n + Nat.2)) + (n + Nat.1)
                    = ((p + p) + (n + Nat.1)) - (n + Nat.2))
                ((p + p - n - Nat.2) + (n + Nat.1)
                    = ((p + p) + (n + Nat.1)) - (n + Nat.2))
                (Nat.1 <= p + p)
                add_sub_comm(p + p, Nat.1, n + Nat.1)
                (Nat.1 <= p + p implies (p + p - Nat.1) + (n + Nat.1)
                    = ((p + p) + (n + Nat.1)) - Nat.1)
                ((p + p - Nat.1) + (n + Nat.1) = ((p + p) + (n + Nat.1)) - Nat.1)
                (((p + p) + (n + Nat.1)) - Nat.1 = p + p + n)
                ((p + p - Nat.1) + (n + Nat.1) = p + p + n)
                ((n + Nat.2) + (p + p - Nat.1) = (p + p - Nat.1) + (n + Nat.2))
                (n + Nat.2 = n + Nat.1 + Nat.1)
                ((p + p - Nat.1) + (n + Nat.2) = (p + p - Nat.1) + (n + Nat.1 + Nat.1))
                add_assoc(p + p - Nat.1, n + Nat.1, Nat.1)
                ((p + p - Nat.1) + (n + Nat.1 + Nat.1) = (p + p - Nat.1) + (n + Nat.1) + Nat.1)
                ((p + p - Nat.1) + (n + Nat.2) = (p + p - Nat.1) + (n + Nat.1) + Nat.1)
                ((p + p - Nat.1) + (n + Nat.1) + Nat.1 = p + p + n + Nat.1)
                ((n + Nat.2) + (p + p - Nat.1) = p + p + n + Nat.1)
                ((p + p) + (n + Nat.1) = p + p + n + Nat.1)
                ((n + Nat.2) + (p + p - Nat.1) = (p + p) + (n + Nat.1))
                add_imp_sub(n + Nat.2, p + p - Nat.1, (p + p) + (n + Nat.1))
                (((p + p) + (n + Nat.1)) - (n + Nat.2) = p + p - Nat.1)
                ((p + p - n - Nat.2) + (n + Nat.1) = p + p - Nat.1)
                add_imp_sub(p + p - n - Nat.2, n + Nat.1, p + p - Nat.1)
                ((p + p - Nat.1) - (p + p - n - Nat.2) = n + Nat.1)
                (n + Nat.1 <= (p + p - Nat.1) - j)
                (p < n + Nat.1)
                lt_and_lte(p, n + Nat.1, p + p - Nat.1 - j)
                (p < p + p - Nat.1 - j)
                (j <= p + p - n - Nat.2)
                (p + p - n - Nat.2 <= p + p - Nat.1)
                lte_trans(j, p + p - n - Nat.2, p + p - Nat.1)
                (j <= p + p - Nat.1)
                sub_lte_sub(p + p - Nat.1, j, Nat.0)
                ((p + p - Nat.1) - j <= (p + p - Nat.1) - Nat.0)
                ((p + p - Nat.1) - Nat.0 = p + p - Nat.1)
                (p + p - Nat.1 - j <= p + p - Nat.1)
                sub_one_lt(p + p)
                (p + p - Nat.1 < p + p)
                lte_and_lt(p + p - Nat.1 - j, p + p - Nat.1, p + p)
                (p + p - Nat.1 - j < p + p)
                not_divides_between_p_and_2p(p, p + p - Nat.1 - j)
                not p.divides(p + p - Nat.1 - j)
                false
            }
            false
        }
        not p.divides(central_binom(n))
    }
}

/// A prime equal to `n` above `2n/3` does not divide the central binomial coefficient.
///
/// The boundary case of the interval: when `p = n`, the falling product `(2n)(2n - 1)...(n + 1)`
/// has `2n = 2p` as its only multiple of `p`, and the same double-counting argument as the
/// strict case applies.
theorem prime_eq_n_two_thirds_not_divides(p: Nat, n: Nat) {
    p.is_prime and Nat.2 < p and p = n and Nat.2 * n < Nat.3 * p
        implies not p.divides(central_binom(n))
} by {
    if p.is_prime and Nat.2 < p and p = n and Nat.2 * n < Nat.3 * p {
        p <= n
        Nat.2 < n
        Nat.1 < n
        Nat.1 <= n
        n != Nat.0
        Nat.0 < n
        sub_one_lt(n)
        (n - Nat.1 < n)
        (n < n + n)
        lt_trans(n - Nat.1, n, n + n)
        (n - Nat.1 < n + n)
        falling_product_eq_binom_mul_factorial(n + n, n - Nat.1)
        (falling_product(n + n, n - Nat.1)
            = (n + n).binom((n - Nat.1).suc) * (n - Nat.1).suc.factorial)
        add_sub(n, Nat.1)
        ((n - Nat.1).suc = n)
        (falling_product(n + n, n - Nat.1) = (n + n).binom(n) * n.factorial)
        central_binom_eq(n)
        (central_binom(n) = (n + n).binom(n))
        (falling_product(n + n, n - Nat.1) = central_binom(n) * n.factorial)
        if p.divides(central_binom(n)) {
            (p.divides(central_binom(n)) = exists(c1: Nat) { p * c1 = central_binom(n) })
            let (c1: Nat) satisfy {
                p * c1 = central_binom(n)
            }
            p != Nat.0
            divides_factorial(p, n)
            (p != Nat.0 and p <= n implies p.divides(n.factorial))
            p.divides(n.factorial)
            (p.divides(n.factorial) = exists(c2: Nat) { p * c2 = n.factorial })
            let (c2: Nat) satisfy {
                p * c2 = n.factorial
            }
            (falling_product(n + n, n - Nat.1) = (p * c1) * (p * c2))
            mul_assoc(p, c1, p * c2)
            ((p * c1) * (p * c2) = p * (c1 * (p * c2)))
            mul_comm(p, c2)
            (c1 * (p * c2) = c1 * (c2 * p))
            mul_assoc(c1, c2, p)
            (c1 * (c2 * p) = (c1 * c2) * p)
            mul_comm(c1 * c2, p)
            ((c1 * c2) * p = p * (c1 * c2))
            ((p * c1) * (p * c2) = p * (p * (c1 * c2)))
            (falling_product(n + n, n - Nat.1) = p * (p * (c1 * c2)))
            // fp = 2n * fp(2n-1, n-2)
            (Nat.0 < n - Nat.1)
            (n - Nat.1 < n + n)
            (Nat.1 + Nat.1 <= n)
            sub_sub_assoc(n, Nat.1, Nat.1)
            (Nat.1 + Nat.1 <= n implies (n - Nat.1) - Nat.1 = n - (Nat.1 + Nat.1))
            ((n - Nat.1) - Nat.1 = n - (Nat.1 + Nat.1))
            (Nat.1 + Nat.1 = Nat.2)
            ((n - Nat.1) - Nat.1 = n - Nat.2)
            falling_product_split(n + n, n - Nat.1, Nat.0)
            (Nat.0 < n - Nat.1 and n - Nat.1 < n + n
                implies falling_product(n + n, n - Nat.1)
                    = falling_product(n + n, Nat.0)
                        * falling_product(n + n - Nat.1, (n - Nat.1) - Nat.0.suc))
            (falling_product(n + n, n - Nat.1)
                = falling_product(n + n, Nat.0)
                    * falling_product(n + n - Nat.1, (n - Nat.1) - Nat.0.suc))
            ((n - Nat.1) - Nat.0.suc = n - Nat.2)
            (falling_product(n + n, n - Nat.1)
                = falling_product(n + n, Nat.0)
                    * falling_product(n + n - Nat.1, n - Nat.2))
            falling_product_zero(n + n)
            (falling_product(n + n, Nat.0) = n + n)
            (falling_product(n + n, n - Nat.1)
                = (n + n) * falling_product(n + n - Nat.1, n - Nat.2))
            (n + n = p + p)
            ((n + n) * falling_product(n + n - Nat.1, n - Nat.2)
                = p * (Nat.2 * falling_product(n + n - Nat.1, n - Nat.2)))
            (falling_product(n + n, n - Nat.1)
                = p * (Nat.2 * falling_product(n + n - Nat.1, n - Nat.2)))
            (p * (p * (c1 * c2))
                = p * (Nat.2 * falling_product(n + n - Nat.1, n - Nat.2)))
            mul_cancel_left(p, p * (c1 * c2), Nat.2 * falling_product(n + n - Nat.1, n - Nat.2))
            (p * (c1 * c2) = Nat.2 * falling_product(n + n - Nat.1, n - Nat.2))
            (p.divides(Nat.2 * falling_product(n + n - Nat.1, n - Nat.2)))
            prime_divides_mul(p, Nat.2, falling_product(n + n - Nat.1, n - Nat.2))
            (p.divides(Nat.2) or p.divides(falling_product(n + n - Nat.1, n - Nat.2)))
            if p.divides(Nat.2) {
                divides_lte(p, Nat.2)
                (Nat.2 = Nat.0 or p <= Nat.2)
                Nat.2 != Nat.0
                p <= Nat.2
                lte_and_lt(p, Nat.2, p)
                p < p
                false
            }
            not p.divides(Nat.2)
            p.divides(falling_product(n + n - Nat.1, n - Nat.2))
            prime_divides_falling_product_factor(n + n - Nat.1, n - Nat.2, p)
            exists(j: Nat) {
                j <= n - Nat.2 and p.divides(n + n - Nat.1 - j)
            }
            let (j: Nat) satisfy {
                j <= n - Nat.2 and p.divides(n + n - Nat.1 - j)
            }
            (j <= n - Nat.2)
            (Nat.0 < n - Nat.1)
            (n - Nat.1 > Nat.0)
            sub_one_lt(n - Nat.1)
            (n - Nat.2 < n - Nat.1)
            (n - Nat.2 <= n - Nat.1)
            sub_one_lt(n)
            (n - Nat.1 < n)
            (n - Nat.1 <= n)
            lte_trans(n - Nat.2, n - Nat.1, n)
            (n - Nat.2 <= n)
            (n < n + n)
            lt_imp_lte_suc(n, n + n)
            (n.suc <= n + n)
            (n.suc = n + Nat.1)
            (n + Nat.1 <= n + n)
            (n + Nat.1 <= n + n) = exists(h2: Nat) { n + Nat.1 + h2 = n + n }
            let (h: Nat) satisfy {
                n + Nat.1 + h = n + n
            }
            add_imp_sub(n + Nat.1, h, n + n)
            (n + Nat.1 + h = n + n)
            (Nat.1 + (n + h) = n + Nat.1 + h)
            add_imp_sub_left(Nat.1, n + h, n + Nat.1 + h)
            ((n + Nat.1 + h) - Nat.1 = n + h)
            ((n + n) - Nat.1 = n + h)
            (n <= n + h)
            (n <= (n + n) - Nat.1)
            lte_trans(n - Nat.2, n, n + n - Nat.1)
            (n - Nat.2 <= n + n - Nat.1)
            sub_lte_sub(n + n - Nat.1, j, n - Nat.2)
            (j <= n - Nat.2 and n - Nat.2 <= n + n - Nat.1
                implies (n + n - Nat.1) - (n - Nat.2) <= (n + n - Nat.1) - j)
            ((n + n - Nat.1) - (n - Nat.2) <= (n + n - Nat.1) - j)
            (Nat.2 <= n)
            add_sub_comm(n, Nat.2, n + Nat.1)
            (Nat.2 <= n implies (n - Nat.2) + (n + Nat.1) = (n + (n + Nat.1)) - Nat.2)
            ((n - Nat.2) + (n + Nat.1) = (n + (n + Nat.1)) - Nat.2)
            (n + (n + Nat.1) = n + n + Nat.1)
            ((n - Nat.2) + (n + Nat.1) = (n + n + Nat.1) - Nat.2)
            (Nat.1 + Nat.1 = Nat.2)
            (Nat.2 < n)
            (n <= n + n)
            lt_imp_lte_suc(n, n + n)
            (n.suc <= n + n)
            (n.suc = n + Nat.1)
            (n + Nat.1 <= n + n)
            (n + Nat.1 <= n + n + Nat.1)
            (Nat.2 <= n)
            lte_trans(Nat.2, n, n + Nat.1)
            (Nat.2 <= n + Nat.1)
            lte_trans(Nat.2, n + Nat.1, n + n + Nat.1)
            (Nat.2 <= n + n + Nat.1)
            (Nat.1 + Nat.1 <= n + n + Nat.1)
            sub_sub_assoc(n + n + Nat.1, Nat.1, Nat.1)
            (Nat.1 + Nat.1 <= n + n + Nat.1
                implies (n + n + Nat.1 - Nat.1) - Nat.1 = (n + n + Nat.1) - (Nat.1 + Nat.1))
            ((n + n + Nat.1 - Nat.1) - Nat.1 = (n + n + Nat.1) - (Nat.1 + Nat.1))
            ((n + n + Nat.1 - Nat.1) - Nat.1 = (n + n + Nat.1) - Nat.2)
            ((n + n) + Nat.1 = n + n + Nat.1)
            add_imp_sub(n + n, Nat.1, n + n + Nat.1)
            ((n + n + Nat.1) - Nat.1 = n + n)
            ((n + n + Nat.1 - Nat.1) - Nat.1 = n + n - Nat.1)
            ((n + n + Nat.1) - Nat.2 = n + n - Nat.1)
            ((n - Nat.2) + (n + Nat.1) = n + n - Nat.1)
            add_imp_sub(n - Nat.2, n + Nat.1, n + n - Nat.1)
            ((n + n - Nat.1) - (n - Nat.2) = n + Nat.1)
            (n + Nat.1 <= (n + n - Nat.1) - j)
            (p < n + Nat.1)
            lt_and_lte(p, n + Nat.1, n + n - Nat.1 - j)
            (p < n + n - Nat.1 - j)
            (j <= n - Nat.2)
            (n - Nat.2 <= n + n - Nat.1)
            lte_trans(j, n - Nat.2, n + n - Nat.1)
            (j <= n + n - Nat.1)
            sub_lte_sub(n + n - Nat.1, j, Nat.0)
            (Nat.0 <= j and j <= n + n - Nat.1
                implies (n + n - Nat.1) - j <= (n + n - Nat.1) - Nat.0)
            ((n + n - Nat.1) - j <= (n + n - Nat.1) - Nat.0)
            ((n + n - Nat.1) - Nat.0 = n + n - Nat.1)
            ((n + n - Nat.1) - j <= n + n - Nat.1)
            Nat.0 < n
            (n < n + n)
            lt_trans(Nat.0, n, n + n)
            Nat.0 < n + n
            (n + n > Nat.0)
            sub_one_lt(n + n)
            (n + n - Nat.1 < n + n)
            (n + n = p + p)
            lte_and_lt((n + n - Nat.1) - j, n + n - Nat.1, n + n)
            ((n + n - Nat.1) - j < n + n)
            ((n + n - Nat.1) - j < p + p)
            not_divides_between_p_and_2p(p, n + n - Nat.1 - j)
            not p.divides(n + n - Nat.1 - j)
            false
        }
        not p.divides(central_binom(n))
    }
}

/// A prime above `2n/3` and at most `n` does not divide the central binomial coefficient.
///
/// The full interval `(2n/3, n]`: both the strict case `p < n` and the boundary case `p = n`
/// are covered by the two arguments above.
theorem prime_above_two_thirds_not_divides(p: Nat, n: Nat) {
    p.is_prime and Nat.2 < p and p <= n and Nat.2 * n < Nat.3 * p
        implies not p.divides(central_binom(n))
} by {
    if p.is_prime and Nat.2 < p and p <= n and Nat.2 * n < Nat.3 * p {
        if p < n {
            prime_lt_n_two_thirds_not_divides(p, n)
            (p.is_prime and Nat.2 < p and p < n and Nat.2 * n < Nat.3 * p
                implies not p.divides(central_binom(n)))
            not p.divides(central_binom(n))
        }
        if not (p < n) {
            not (p < n)
            lt_or_lte(p, n)
            (p < n or n <= p)
            n <= p
            lte_antisymm(p, n)
            p = n
            prime_eq_n_two_thirds_not_divides(p, n)
            (p.is_prime and Nat.2 < p and p = n and Nat.2 * n < Nat.3 * p
                implies not p.divides(central_binom(n)))
            not p.divides(central_binom(n))
        }
        not p.divides(central_binom(n))
    }
}

/// A prime factor of the central binomial coefficient is at most `2n`.
///
/// The coefficient divides `(2n)!`, and a prime dividing a factorial is at most the argument.
theorem prime_factor_of_central_binom_at_most_2n(p: Nat, n: Nat) {
    p.is_prime and p.divides(central_binom(n)) implies p <= n + n
} by {
    if p.is_prime and p.divides(central_binom(n)) {
        choose_add(n, n)
        ((n + n).binom(n) * n.factorial * n.factorial = (n + n).factorial)
        central_binom_eq(n)
        (central_binom(n) = (n + n).binom(n))
        (central_binom(n) * n.factorial * n.factorial = (n + n).factorial)
        (central_binom(n) * (n.factorial * n.factorial) = (n + n).factorial)
        central_binom(n).divides((n + n).factorial)
        divides_trans(p, central_binom(n), (n + n).factorial)
        p.divides((n + n).factorial)
        prime_divides_factorial_imp_lte(p, n + n)
        (p.is_prime and p.divides((n + n).factorial) implies p <= n + n)
        p <= n + n
    }
}

/// Under no prime in `(n, 2n]`, a prime factor of the central binomial coefficient is at most `n`.
///
/// The coefficient's prime factors all lie in `[1, 2n]`; those above `n` would be primes in the
/// forbidden interval.
theorem prime_factor_of_central_binom_at_most_n(p: Nat, n: Nat) {
    p.is_prime and p.divides(central_binom(n))
        and (forall(q: Nat) {
            q.is_prime and n < q and q <= n + n implies not q.divides(central_binom(n))
        }) implies p <= n
} by {
    if p.is_prime and p.divides(central_binom(n))
        and (forall(q: Nat) {
            q.is_prime and n < q and q <= n + n implies not q.divides(central_binom(n))
        }) {
        prime_factor_of_central_binom_at_most_2n(p, n)
        (p.is_prime and p.divides(central_binom(n)) implies p <= n + n)
        p <= n + n
        if n < p {
            (p.is_prime and n < p and p <= n + n)
            (forall(q: Nat) {
                q.is_prime and n < q and q <= n + n implies not q.divides(central_binom(n))
            })
            (p.is_prime and n < p and p <= n + n implies not p.divides(central_binom(n)))
            not p.divides(central_binom(n))
            false
        }
        not (n < p)
        lt_or_lte(n, p)
        (n < p or p <= n)
        p <= n
    }
}

/// Under no prime in `(n, 2n]`, a prime factor of the central binomial coefficient is at most
/// `2n/3`.
///
/// The valuation lemma: a prime in `(2n/3, n]` never divides the coefficient, so the factor
/// must sit below `2n/3`.
theorem prime_factor_of_central_binom_two_thirds(p: Nat, n: Nat) {
    Nat.2 < n and p.is_prime and p.divides(central_binom(n))
        and (forall(q: Nat) {
            q.is_prime and n < q and q <= n + n implies not q.divides(central_binom(n))
        }) implies Nat.3 * p <= n + n
} by {
    if Nat.2 < n and p.is_prime and p.divides(central_binom(n))
        and (forall(q: Nat) {
            q.is_prime and n < q and q <= n + n implies not q.divides(central_binom(n))
        }) {
        prime_factor_of_central_binom_at_most_n(p, n)
        (p.is_prime and p.divides(central_binom(n))
            and (forall(q: Nat) {
                q.is_prime and n < q and q <= n + n implies not q.divides(central_binom(n))
            }) implies p <= n)
        p <= n
        if n + n < Nat.3 * p {
            if p = Nat.2 {
                (Nat.2 * n < Nat.3 * Nat.2)
                (Nat.3 * Nat.2 = Nat.6)
                (Nat.2 * n < Nat.6)
                (Nat.6 = Nat.2 * Nat.3)
                (Nat.2 * n < Nat.2 * Nat.3)
                Nat.2 != Nat.0
                lt_cancel_mul(Nat.2, n, Nat.3)
                n < Nat.3
                lt_imp_lte_suc(Nat.2, n)
                (Nat.2.suc <= n)
                (Nat.2.suc = Nat.3)
                Nat.3 <= n
                lt_and_lte(n, Nat.3, n)
                n < n
                false
            }
            p != Nat.2
            (p.is_prime = (Nat.1 < p and not p.is_composite))
            Nat.1 < p
            lt_imp_lte_suc(Nat.1, p)
            Nat.2 <= p
            (Nat.2 <= p and Nat.2 != p)
            Nat.2 < p
            prime_above_two_thirds_not_divides(p, n)
            (p.is_prime and Nat.2 < p and p <= n and Nat.2 * n < Nat.3 * p
                implies not p.divides(central_binom(n)))
            not p.divides(central_binom(n))
            false
        }
        not (n + n < Nat.3 * p)
        lt_or_lte(n + n, Nat.3 * p)
        (n + n < Nat.3 * p or Nat.3 * p <= n + n)
        Nat.3 * p <= n + n
    }
}

// Bertrand's postulate, left commented out: the assembly below is complete up to the
// analytic inequality, which needs the valuation power bound, the square-root split, and a
// large-`n` inequality that this library does not yet contain.
//
// The plan, with the pieces that exist:
//
//   1. `central_binom_lower_bound`: 4^n <= (2n + 1) * C(2n, n).          [done]
//   2. `prime_above_two_thirds_not_divides`: a prime in (2n/3, n] never
//      divides C(2n, n).                                                [done]
//   3. `primorial_lte_four_pow` (nat_primorial_bound): the product of the
//      primes up to x is at most 4^x.                                   [done]
//   4. `prime_factor_of_central_binom_two_thirds`: under no prime in
//      (n, 2n], every prime factor of C(2n, n) is at most 2n/3.         [done]
//   5. Missing: each prime power p^(a_p) in C(2n, n) is at most 2n
//      (a_p is bounded by the number of powers of p below 2n).
//   6. Missing: splitting the primes at sqrt(2n) — those above it divide
//      C(2n, n) at most once — gives
//          C(2n, n) <= (2n)^sqrt(2n) * P(2n/3) <= (2n)^sqrt(2n) * 4^(2n/3),
//      and needs the count of primes at most sqrt(2n) bounded by sqrt(2n).
//   7. Missing: the analytic inequality
//          (2n + 1) * (2n)^sqrt(2n) * 4^(2n/3) <= 4^n  for n >= n0,
//      which contradicts 4^n <= (2n + 1) * C(2n, n) together with (6) under
//      the no-prime assumption. This is the step that needs real-analysis
//      style estimates and is the current blocker.
//   8. Missing: the finitely many cases n < n0, covered by a descending
//      list of primes (each at most twice the next) with primality proofs.
//
// /// Every integer at least two has a prime strictly between it and its double.
// ///
// /// Bertrand's postulate (Chebyshev's theorem), proved by the Erdős route: the central
// /// binomial coefficient is too large to be built only from primes at most `2n/3`.
// theorem bertrand_postulate(n: Nat) {
//     Nat.2 <= n implies exists(p: Nat) {
//         p.is_prime and n < p and p < n + n
//     }
// } by {
//     if Nat.2 <= n {
//         // n = 2: 3 works. n >= 3: assume no prime in (n, 2n], split on the
//         // parity of n, and derive the contradiction from steps 1-7 above.
//         // For the no-prime assumption, first note 2n itself is composite for
//         // n >= 2, so a prime at most 2n is automatically below 2n.
//         exists(p: Nat) {
//             p.is_prime and n < p and p < n + n
//         }
//     }
// }
