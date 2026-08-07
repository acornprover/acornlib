from nat import Nat, lte_trans, lte_mul_both
from data.basic.functions import function_extensionality
from data.nat.nat_range_prod import range_prod, range_prod_zero, range_prod_suc, range_prod_split,
    prod_shift_fn
from data.nat.nat_prime_interval_product import prime_or_one, prime_or_one_prime,
    prime_or_one_composite, interval_prime_fn

numerals Nat

/// The product of the primes at most `n`.
///
/// Taken over the whole range rather than over a list of primes, with `prime_or_one` contributing
/// a factor of one at every composite position. That is the same device the interval products use,
/// and it means nothing has to be enumerated.
define primorial(n: Nat) -> Nat {
    range_prod(prime_or_one, n.suc)
}

/// The primorial of zero is one.
theorem primorial_zero {
    primorial(Nat.0) = Nat.1
} by {
    (primorial(Nat.0) = range_prod(prime_or_one, Nat.0.suc))
    range_prod_suc(prime_or_one, Nat.0)
    (range_prod(prime_or_one, Nat.0.suc) = range_prod(prime_or_one, Nat.0)
        * prime_or_one(Nat.0))
    range_prod_zero(prime_or_one)
    (range_prod(prime_or_one, Nat.0) = Nat.1)
    not (Nat.1 < Nat.0)
    not Nat.0.is_prime
    prime_or_one_composite(Nat.0)
    (prime_or_one(Nat.0) = Nat.1)
    primorial(Nat.0) = Nat.1
}

/// Each step multiplies by the next position.
theorem primorial_suc(n: Nat) {
    primorial(n.suc) = primorial(n) * prime_or_one(n.suc)
} by {
    (primorial(n.suc) = range_prod(prime_or_one, n.suc.suc))
    range_prod_suc(prime_or_one, n.suc)
    (range_prod(prime_or_one, n.suc.suc)
        = range_prod(prime_or_one, n.suc) * prime_or_one(n.suc))
    (primorial(n) = range_prod(prime_or_one, n.suc))
    primorial(n.suc) = primorial(n) * prime_or_one(n.suc)
}

/// A position contributes at most itself.
///
/// At a prime it contributes the prime, elsewhere one, and both are at most the position once the
/// position is positive.
theorem prime_or_one_lte(x: Nat) {
    Nat.1 <= x implies prime_or_one(x) <= x
} by {
    if Nat.1 <= x {
        if x.is_prime {
            prime_or_one_prime(x)
            (prime_or_one(x) = x)
            prime_or_one(x) <= x
        }
        if not x.is_prime {
            prime_or_one_composite(x)
            (prime_or_one(x) = Nat.1)
            prime_or_one(x) <= x
        }
        prime_or_one(x) <= x
    }
}

/// The shifted factor of a range product is the factor of the interval starting there.
theorem prod_shift_prime_or_one(a: Nat) {
    prod_shift_fn(prime_or_one, a) = interval_prime_fn(a)
} by {
    forall(i: Nat) {
        (prod_shift_fn(prime_or_one, a, i) = prime_or_one(a + i))
        (interval_prime_fn(a, i) = prime_or_one(a + i))
        prod_shift_fn(prime_or_one, a)(i) = interval_prime_fn(a)(i)
    }
    function_extensionality[Nat, Nat](prod_shift_fn(prime_or_one, a), interval_prime_fn(a))
    prod_shift_fn(prime_or_one, a) = interval_prime_fn(a)
}

/// An odd primorial splits at the midpoint.
///
/// The primes up to `2m + 1` are those up to `m + 1` together with those strictly above it, and
/// the second group is exactly the interval the odd binomial bound covers.
theorem primorial_odd_split(m: Nat) {
    (primorial(m + m.suc)
        = primorial(m.suc) * range_prod(interval_prime_fn(m.suc.suc), m))
} by {
    ((m + m.suc).suc = m.suc.suc + m)
    (primorial(m + m.suc) = range_prod(prime_or_one, (m + m.suc).suc))
    (primorial(m + m.suc) = range_prod(prime_or_one, m.suc.suc + m))
    range_prod_split(prime_or_one, m.suc.suc, m)
    (range_prod(prime_or_one, m.suc.suc + m)
        = range_prod(prime_or_one, m.suc.suc)
            * range_prod(prod_shift_fn(prime_or_one, m.suc.suc), m))
    (primorial(m.suc) = range_prod(prime_or_one, m.suc.suc))
    prod_shift_prime_or_one(m.suc.suc)
    (prod_shift_fn(prime_or_one, m.suc.suc) = interval_prime_fn(m.suc.suc))
    (primorial(m + m.suc)
        = primorial(m.suc) * range_prod(interval_prime_fn(m.suc.suc), m))
}

/// The primorial of one is one.
theorem primorial_one {
    primorial(Nat.1) = Nat.1
} by {
    (Nat.0.suc = Nat.1)
    primorial_suc(Nat.0)
    (primorial(Nat.0.suc) = primorial(Nat.0) * prime_or_one(Nat.0.suc))
    primorial_zero
    (primorial(Nat.0) = Nat.1)
    not (Nat.1 < Nat.1)
    not Nat.1.is_prime
    prime_or_one_composite(Nat.1)
    (prime_or_one(Nat.1) = Nat.1)
    (Nat.1 * Nat.1 = Nat.1)
    primorial(Nat.1) = Nat.1
}

/// A composite position contributes nothing.
///
/// Stated at a successor rather than through a predecessor, since `Nat` has no predecessor.
theorem primorial_composite_step(j: Nat) {
    j.suc.is_composite implies primorial(j.suc) = primorial(j)
} by {
    if j.suc.is_composite {
        not j.suc.is_prime
        prime_or_one_composite(j.suc)
        (prime_or_one(j.suc) = Nat.1)
        primorial_suc(j)
        (primorial(j.suc) = primorial(j) * prime_or_one(j.suc))
        (primorial(j) * Nat.1 = primorial(j))
        primorial(j.suc) = primorial(j)
    }
}

/// Twice anything above one is composite.
theorem double_is_composite(t: Nat) {
    Nat.1 < t implies (t + t).is_composite
} by {
    if Nat.1 < t {
        Nat.1 < Nat.2
        (t + t = Nat.2 * t)
        exists(b: Nat, c: Nat) {
            Nat.1 < b and Nat.1 < c and t + t = b * c
        }
        ((t + t).is_composite = exists(b: Nat, c: Nat) {
            Nat.1 < b and Nat.1 < c and t + t = b * c
        })
        (t + t).is_composite
    }
}
