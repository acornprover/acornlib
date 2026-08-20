from nat import Nat
from real import Real, converges_to
from data.nat.nat_density import density_seq, has_natural_density, has_natural_density_apply
from analysis import is_bounded_above, limsup, is_bounded_below, liminf,
    limsup_eq_limit, liminf_eq_limit
from data.nat.nat_density_extremes import upper_density, lower_density, density_seq_bounded_above,
    density_seq_bounded_below

/// The upper density of a set with a natural density is that density.
theorem upper_density_of_natural_density(p: Nat -> Bool, d: Real) {
    has_natural_density(p, d) implies upper_density(p) = d
} by {
    if has_natural_density(p, d) {
        has_natural_density_apply(p, d)
        converges_to(density_seq(p), d)
        density_seq_bounded_above(p)
        is_bounded_above(density_seq(p))
        density_seq_bounded_below(p)
        is_bounded_below(density_seq(p))
        limsup_eq_limit(density_seq(p), d)
        limsup(density_seq(p)) = d
        (upper_density(p) = limsup(density_seq(p)))
        upper_density(p) = d
    }
}

/// The lower density of a set with a natural density is that density.
theorem lower_density_of_natural_density(p: Nat -> Bool, d: Real) {
    has_natural_density(p, d) implies lower_density(p) = d
} by {
    if has_natural_density(p, d) {
        has_natural_density_apply(p, d)
        converges_to(density_seq(p), d)
        density_seq_bounded_above(p)
        is_bounded_above(density_seq(p))
        density_seq_bounded_below(p)
        is_bounded_below(density_seq(p))
        liminf_eq_limit(density_seq(p), d)
        liminf(density_seq(p)) = d
        (lower_density(p) = liminf(density_seq(p)))
        lower_density(p) = d
    }
}

/// A set with a natural density has upper and lower density equal to it.
///
/// The two one-sided densities always exist; this says they carry no more information than the
/// natural density wherever that exists, which is what makes them the right generalisation.
theorem densities_agree_of_natural_density(p: Nat -> Bool, d: Real) {
    has_natural_density(p, d)
        implies lower_density(p) = d and upper_density(p) = d
} by {
    if has_natural_density(p, d) {
        lower_density_of_natural_density(p, d)
        lower_density(p) = d
        upper_density_of_natural_density(p, d)
        upper_density(p) = d
        (lower_density(p) = d and upper_density(p) = d)
    }
}
