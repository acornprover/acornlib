from nat import Nat, from_nat
from real import Real, abs_neg
from algebra.add_comm_monoid_rearrange import add_swap_inner
from data.nat.nat_counting_function import count_upto, count_upto_zero, count_upto_suc_true,
    count_upto_suc_false
from data.nat.nat_density import range_size, range_size_positive, density_seq

numerals Nat

/// The complement of a predicate on the naturals.
define not_pred(p: Nat -> Bool) -> (Nat -> Bool) {
    function(i: Nat) {
        not p(i)
    }
}

/// A natural satisfies the complement exactly when it fails the predicate.
theorem not_pred_apply(p: Nat -> Bool, i: Nat) {
    not_pred(p)(i) = not p(i)
} by {
    not_pred(p)(i) = not p(i)
}

/// The counts of a predicate and its complement add up to the size of the range.
///
/// Each step of the recurrence increments exactly one of the two counts, so their sum tracks
/// the index. This is the counting identity behind the density of a complement.
theorem count_upto_complement(p: Nat -> Bool, n: Nat) {
    count_upto(not_pred(p), n) + count_upto(p, n) = n
} by {
    define q(x: Nat) -> Bool {
        count_upto(not_pred(p), x) + count_upto(p, x) = x
    }
    count_upto_zero(not_pred(p))
    count_upto(not_pred(p), Nat.0) = Nat.0
    count_upto_zero(p)
    count_upto(p, Nat.0) = Nat.0
    q(Nat.0)
    forall(k: Nat) {
        if q(k) {
            count_upto(not_pred(p), k) + count_upto(p, k) = k
            not_pred_apply(p, k)
            (not_pred(p)(k) = not p(k))
            if p(k) {
                count_upto_suc_true(p, k)
                count_upto(p, k.suc) = count_upto(p, k) + Nat.1
                not not_pred(p)(k)
                count_upto_suc_false(not_pred(p), k)
                count_upto(not_pred(p), k.suc) = count_upto(not_pred(p), k)
                (count_upto(not_pred(p), k.suc) + count_upto(p, k.suc)
                    = count_upto(not_pred(p), k) + (count_upto(p, k) + Nat.1))
                (count_upto(not_pred(p), k) + (count_upto(p, k) + Nat.1)
                    = (count_upto(not_pred(p), k) + count_upto(p, k)) + Nat.1)
                count_upto(not_pred(p), k.suc) + count_upto(p, k.suc) = k + Nat.1
                k + Nat.1 = k.suc
                count_upto(not_pred(p), k.suc) + count_upto(p, k.suc) = k.suc
            }
            if not p(k) {
                count_upto_suc_false(p, k)
                count_upto(p, k.suc) = count_upto(p, k)
                not_pred(p)(k)
                count_upto_suc_true(not_pred(p), k)
                count_upto(not_pred(p), k.suc) = count_upto(not_pred(p), k) + Nat.1
                (count_upto(not_pred(p), k.suc) + count_upto(p, k.suc)
                    = (count_upto(not_pred(p), k) + Nat.1) + count_upto(p, k))
                ((count_upto(not_pred(p), k) + Nat.1) + count_upto(p, k)
                    = (count_upto(not_pred(p), k) + count_upto(p, k)) + Nat.1)
                count_upto(not_pred(p), k.suc) + count_upto(p, k.suc) = k + Nat.1
                k + Nat.1 = k.suc
                count_upto(not_pred(p), k.suc) + count_upto(p, k.suc) = k.suc
            }
            q(k.suc)
        }
        (q(k) implies q(k.suc))
    }
    q(Nat.0) and forall(k: Nat) {
        q(k) implies q(k.suc)
    }
    Nat.induction(q)
    q(n)
}

/// The density fractions of a predicate and its complement add up to one.
theorem density_seq_complement(p: Nat -> Bool, n: Nat) {
    density_seq(not_pred(p), n) = Real.1 - density_seq(p, n)
} by {
    count_upto_complement(p, n.suc)
    count_upto(not_pred(p), n.suc) + count_upto(p, n.suc) = n.suc
    (from_nat[Real](count_upto(not_pred(p), n.suc) + count_upto(p, n.suc))
        = from_nat[Real](count_upto(not_pred(p), n.suc))
            + from_nat[Real](count_upto(p, n.suc)))
    (from_nat[Real](count_upto(not_pred(p), n.suc)) + from_nat[Real](count_upto(p, n.suc))
        = from_nat[Real](n.suc))
    range_size(n) = from_nat[Real](n.suc)
    (from_nat[Real](count_upto(not_pred(p), n.suc))
        = range_size(n) - from_nat[Real](count_upto(p, n.suc)))
    range_size_positive(n)
    Real.0 < range_size(n)
    range_size(n) != Real.0
    (density_seq(not_pred(p), n)
        = from_nat[Real](count_upto(not_pred(p), n.suc)) / range_size(n))
    (density_seq(not_pred(p), n)
        = (range_size(n) - from_nat[Real](count_upto(p, n.suc))) / range_size(n))
    ((range_size(n) - from_nat[Real](count_upto(p, n.suc))) / range_size(n)
        = range_size(n) / range_size(n)
            - from_nat[Real](count_upto(p, n.suc)) / range_size(n))
    range_size(n) / range_size(n) = Real.1
    (density_seq(p, n) = from_nat[Real](count_upto(p, n.suc)) / range_size(n))
    density_seq(not_pred(p), n) = Real.1 - density_seq(p, n)
}

/// A number is close to a limit exactly when its reflection is close to the reflected limit.
theorem one_sub_is_close(x: Real, y: Real, eps: Real) {
    x.is_close(y, eps) implies (Real.1 - x).is_close(Real.1 - y, eps)
} by {
    if x.is_close(y, eps) {
        (x.is_close(y, eps) = ((x - y).abs < eps))
        (x - y).abs < eps
        (Real.1 - x = Real.1 + -x)
        (Real.1 - y = Real.1 + -y)
        (-(Real.1 + -y) = -Real.1 + y)
        ((Real.1 - x) - (Real.1 - y) = (Real.1 + -x) + (-Real.1 + y))
        add_swap_inner[Real](Real.1, -x, -Real.1, y)
        ((Real.1 + -x) + (-Real.1 + y) = (Real.1 + -Real.1) + (-x + y))
        (Real.1 + -Real.1 = Real.0)
        (Real.0 + (-x + y) = -x + y)
        ((Real.1 - x) - (Real.1 - y) = -x + y)
        (-x + y = y - x)
        ((Real.1 - x) - (Real.1 - y) = y - x)
        (y - x = -(x - y))
        ((Real.1 - x) - (Real.1 - y) = -(x - y))
        abs_neg(x - y)
        (-(x - y)).abs = (x - y).abs
        ((Real.1 - x) - (Real.1 - y)).abs = (x - y).abs
        ((Real.1 - x) - (Real.1 - y)).abs < eps
        ((Real.1 - x).is_close(Real.1 - y, eps)
            = (((Real.1 - x) - (Real.1 - y)).abs < eps))
        (Real.1 - x).is_close(Real.1 - y, eps)
    }
}

/// The complement's density sequence is close to `1 - d` whenever the original is close to `d`.
///
/// Everything the density of a complement needs except the last step. Reflection moves a point
/// by the same distance it moves the limit, so the same tail bound serves for both sequences.
///
/// Assembling these into `converges_to(density_seq(not_pred(p)), 1 - d)` is what remains. The
/// obstruction is not the mathematics but the citation: `converges_to_intro` will not
/// instantiate at the partially applied `density_seq(not_pred(p))`, even with the hypothesis
/// stated in exactly the form the theorem takes it.
theorem density_seq_complement_is_close(p: Nat -> Bool, d: Real, eps: Real, i: Nat) {
    density_seq(p, i).is_close(d, eps)
        implies density_seq(not_pred(p), i).is_close(Real.1 - d, eps)
} by {
    if density_seq(p, i).is_close(d, eps) {
        one_sub_is_close(density_seq(p, i), d, eps)
        (Real.1 - density_seq(p, i)).is_close(Real.1 - d, eps)
        density_seq_complement(p, i)
        density_seq(not_pred(p), i) = Real.1 - density_seq(p, i)
        density_seq(not_pred(p), i).is_close(Real.1 - d, eps)
    }
}
