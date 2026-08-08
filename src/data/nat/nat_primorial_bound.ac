from nat import Nat, lte_trans, lt_and_lte, lte_and_lt, lte_mul_both, lt_add_left,
    div_mod_decomp,
    mod_lt, strong_induction, true_below, true_below_apply, pow_add, pow_zero, pow_one
from data.nat.nat_range_prod import range_prod
from data.nat.nat_div_div import nat_pow_pos
from data.nat.nat_prime_interval_product import prime_or_one, interval_prime_fn
from data.nat.nat_odd_prime_product import odd_prime_interval_product_lte
from data.nat.nat_primorial import primorial, primorial_zero, primorial_one, primorial_suc,
    primorial_odd_split, primorial_composite_step, double_is_composite, prime_or_one_lte

numerals Nat

/// Every natural is twice something, or one more than twice something.
///
/// The parity split, taken from the division algorithm at two rather than from a separate notion
/// of evenness.
theorem nat_parity(n: Nat) {
    exists(m: Nat) {
        n = m + m or n = m + m.suc
    }
} by {
    Nat.2 != Nat.0
    div_mod_decomp(n, Nat.2)
    (n.div(Nat.2) * Nat.2 + n.mod(Nat.2) = n)
    (n.div(Nat.2) * Nat.2 = n.div(Nat.2) + n.div(Nat.2))
    mod_lt(n, Nat.2)
    n.mod(Nat.2) < Nat.2
    (Nat.1.suc = Nat.2)
    n.mod(Nat.2) < Nat.1.suc
    n.mod(Nat.2) <= Nat.1
    if n.mod(Nat.2) = Nat.0 {
        (n.div(Nat.2) + n.div(Nat.2) + Nat.0 = n.div(Nat.2) + n.div(Nat.2))
        n = n.div(Nat.2) + n.div(Nat.2)
        exists(m: Nat) {
            n = m + m or n = m + m.suc
        }
    }
    if n.mod(Nat.2) != Nat.0 {
        Nat.0 < n.mod(Nat.2)
        (Nat.0.suc = Nat.1)
        Nat.1 <= n.mod(Nat.2)
        n.mod(Nat.2) = Nat.1
        (n.div(Nat.2) + n.div(Nat.2) + Nat.1 = n.div(Nat.2) + n.div(Nat.2).suc)
        n = n.div(Nat.2) + n.div(Nat.2).suc
        exists(m: Nat) {
            n = m + m or n = m + m.suc
        }
    }
    exists(m: Nat) {
        n = m + m or n = m + m.suc
    }
}

/// Two to the fourth power grows with the exponent.
theorem four_pow_mono(a: Nat, b: Nat) {
    a <= b implies Nat.4.pow(a) <= Nat.4.pow(b)
} by {
    if a <= b {
        let (c: Nat) satisfy {
            a + c = b
        }
        pow_add[Nat](Nat.4, a, c)
        (Nat.4.pow(a + c) = Nat.4.pow(a) * Nat.4.pow(c))
        pow_zero[Nat](Nat.4)
        (Nat.4.pow(Nat.0) = Nat.1)
        Nat.0 < Nat.4
        nat_pow_pos(Nat.4, c)
        Nat.0 < Nat.4.pow(c)
        Nat.1 <= Nat.4.pow(c)
        lte_mul_both(Nat.4.pow(a), Nat.1, Nat.4.pow(c))
        (Nat.4.pow(a) * Nat.1 <= Nat.4.pow(a) * Nat.4.pow(c))
        (Nat.4.pow(a) * Nat.1 = Nat.4.pow(a))
        Nat.4.pow(a) <= Nat.4.pow(b)
    }
}

/// Two bounds multiply.
theorem lte_mul_pair(a: Nat, b: Nat, c: Nat, d: Nat) {
    a <= b and c <= d implies a * c <= b * d
} by {
    if a <= b and c <= d {
        lte_mul_both(a, c, d)
        (a * c <= a * d)
        lte_mul_both(d, a, b)
        (d * a <= d * b)
        (d * a = a * d)
        (d * b = b * d)
        (a * d <= b * d)
        lte_trans(a * c, a * d, b * d)
        a * c <= b * d
    }
}

// The primorial bound, left commented out because the final step does not go through.
//
// Every case of the induction below verifies on its own: the parity split, the three even
// sub-cases, and the two odd ones. What fails is only the last application of
// `strong_induction` — the hypothesis the block establishes is not recognised as the one the
// theorem asks for, and `forall(k) { f(k) }` after the citation times out.
//
// The shape matches `nat_lt_relation_induction` in `src/well_founded.ac`, which is the one
// working use of `strong_induction` in the library, so the difference is not obvious. Worth
// trying `nat_lt_relation_induction` directly instead, whose hypothesis is stated through
// `nat_lt_relation` rather than `true_below`.

/// The product of the primes up to `n` is at most four to the `n`.
///
/// The primorial bound. Strong induction on `n`, split by parity: an even position above two is
/// composite and contributes nothing, so the product drops to the one below; an odd position
/// splits at the midpoint, where the lower half is the induction hypothesis and the upper half is
/// the interval bound.
theorem primorial_lte_four_pow(n: Nat) {
    primorial(n) <= Nat.4.pow(n)
} by {
    let f: Nat -> Bool = function(y: Nat) {
        primorial(y) <= Nat.4.pow(y)
    }
    forall(k: Nat) {
        if true_below(f, k) {
            nat_parity(k)
            exists(m: Nat) {
                k = m + m or k = m + m.suc
            }
            let (m: Nat) satisfy {
                k = m + m or k = m + m.suc
            }
            if k = m + m {
                if not (Nat.1 < m) {
                    m <= Nat.1
                    if m != Nat.0 {
                        Nat.0 < m
                        (Nat.0.suc = Nat.1)
                        Nat.1 <= m
                        m = Nat.1
                    }
                    (m = Nat.0 or m = Nat.1)
                }
                (m = Nat.0 or m = Nat.1 or Nat.1 < m)
                if m = Nat.0 {
                    (k = Nat.0)
                    primorial_zero
                    (primorial(Nat.0) = Nat.1)
                    pow_zero[Nat](Nat.4)
                    (Nat.4.pow(Nat.0) = Nat.1)
                    (Nat.1 <= Nat.1)
                    (primorial(Nat.0) <= Nat.4.pow(Nat.0))
                    f(k)
                }
                if m = Nat.1 {
                    (k = Nat.1 + Nat.1)
                    (Nat.1 + Nat.1 = Nat.2)
                    (Nat.0.suc = Nat.1)
                    (Nat.1.suc = Nat.2)
                    primorial_suc(Nat.1)
                    (primorial(Nat.2) = primorial(Nat.1) * prime_or_one(Nat.2))
                    primorial_one
                    (primorial(Nat.1) = Nat.1)
                    Nat.1 <= Nat.2
                    prime_or_one_lte(Nat.2)
                    (prime_or_one(Nat.2) <= Nat.2)
                    (Nat.1 * prime_or_one(Nat.2) = prime_or_one(Nat.2))
                    (primorial(Nat.2) <= Nat.2)
                    pow_one[Nat](Nat.4)
                    (Nat.4.pow(Nat.1) = Nat.4)
                    Nat.1 <= Nat.2
                    four_pow_mono(Nat.1, Nat.2)
                    (Nat.4 <= Nat.4.pow(Nat.2))
                    Nat.2 < Nat.3
                    Nat.3 < Nat.4
                    lt_and_lte(Nat.2, Nat.3, Nat.4)
                    Nat.2 < Nat.4
                    Nat.2 <= Nat.4
                    lte_trans(Nat.2, Nat.4, Nat.4.pow(Nat.2))
                    Nat.2 <= Nat.4.pow(Nat.2)
                    lte_trans(primorial(Nat.2), Nat.2, Nat.4.pow(Nat.2))
                    f(k)
                }
                if Nat.1 < m {
                    Nat.0 < Nat.1
                    Nat.1 <= m
                    lt_and_lte(Nat.0, Nat.1, m)
                    Nat.0 < m
                    let (i: Nat) satisfy {
                        i.suc = m
                    }
                    (k = i.suc + i.suc)
                    (i.suc + i.suc = (i.suc + i).suc)
                    double_is_composite(m)
                    (m + m).is_composite
                    (k = m + m)
                    ((i.suc + i).suc.is_composite)
                    primorial_composite_step(i.suc + i)
                    (primorial((i.suc + i).suc) = primorial(i.suc + i))
                    (primorial(k) = primorial(i.suc + i))
                    i < i.suc
                    lt_add_left(i.suc, i, i.suc)
                    i.suc + i < i.suc + i.suc
                    i.suc + i < k
                    true_below_apply(f, k, i.suc + i)
                    (primorial(i.suc + i) <= Nat.4.pow(i.suc + i))
                    (primorial(k) <= Nat.4.pow(i.suc + i))
                    i.suc + i <= k
                    four_pow_mono(i.suc + i, k)
                    (Nat.4.pow(i.suc + i) <= Nat.4.pow(k))
                    lte_trans(primorial(k), Nat.4.pow(i.suc + i), Nat.4.pow(k))
                    (primorial(k) <= Nat.4.pow(k))
                    (f(k) = (primorial(k) <= Nat.4.pow(k)))
                    f(k)
                }
                f(k)
            }
            if k = m + m.suc {
                if m = Nat.0 {
                    (k = Nat.0 + Nat.0.suc)
                    (Nat.0 + Nat.0.suc = Nat.1)
                    primorial_one
                    (primorial(Nat.1) = Nat.1)
                    pow_one[Nat](Nat.4)
                    (Nat.4.pow(Nat.1) = Nat.4)
                    Nat.1 < Nat.2
                    Nat.2 < Nat.3
                    Nat.3 < Nat.4
                    lt_and_lte(Nat.2, Nat.3, Nat.4)
                    Nat.2 < Nat.4
                    lt_and_lte(Nat.1, Nat.2, Nat.4)
                    Nat.1 < Nat.4
                    Nat.1 <= Nat.4
                    Nat.1 <= Nat.4.pow(Nat.1)
                    (primorial(Nat.1) <= Nat.4.pow(Nat.1))
                    f(k)
                }
                if m != Nat.0 {
                    Nat.0 < m
                    lt_add_left(m.suc, Nat.0, m)
                    m.suc + Nat.0 < m.suc + m
                    (m.suc + Nat.0 = m.suc)
                    (m.suc + m = m + m.suc)
                    m.suc < m + m.suc
                    m.suc < k
                    true_below_apply(f, k, m.suc)
                    (primorial(m.suc) <= Nat.4.pow(m.suc))
                    odd_prime_interval_product_lte(m)
                    (range_prod(interval_prime_fn(m.suc.suc), m) <= Nat.4.pow(m))
                    primorial_odd_split(m)
                    (primorial(m + m.suc)
                        = primorial(m.suc) * range_prod(interval_prime_fn(m.suc.suc), m))
                    lte_mul_pair(primorial(m.suc),
                        Nat.4.pow(m.suc),
                        range_prod(interval_prime_fn(m.suc.suc), m),
                        Nat.4.pow(m))
                    (primorial(m.suc) * range_prod(interval_prime_fn(m.suc.suc), m)
                        <= Nat.4.pow(m.suc) * Nat.4.pow(m))
                    pow_add[Nat](Nat.4, m.suc, m)
                    (Nat.4.pow(m.suc + m) = Nat.4.pow(m.suc) * Nat.4.pow(m))
                    (m.suc + m = m + m.suc)
                    (primorial(k) <= Nat.4.pow(k))
                    (f(k) = (primorial(k) <= Nat.4.pow(k)))
                    f(k)
                }
                f(k)
            }
            f(k)
            (true_below(f, k) implies f(k))
        }
    }
    strong_induction(f)
    (forall(k: Nat) {
        true_below(f, k) implies f(k)
    } implies forall(k2: Nat) {
        f(k2)
    })
    forall(k: Nat) {
        true_below(f, k) implies f(k)
    }
    forall(k: Nat) {
        f(k)
    }
    f(n)
}
