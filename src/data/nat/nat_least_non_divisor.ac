from nat import Nat, divides_lte, divides_self, gcd_mul_lcm, lt_or_lte, lte_mul_both,
    lte_and_lt, lt_and_lte, lte_antisymm, lt_trans, lt_imp_lte_suc, lte_cancel_suc,
    only_zero_lte_zero
from number_theory import lcm_divides_of_common
from data.nat.nat_least_failure import is_least_failure, is_least_failure_fails,
    is_least_failure_holds_below, is_least_failure_intro, least_failure_exists,
    least_failure_unique, least_failure_lower_bound

numerals Nat

/// True if `m` is zero or divides `n`.
///
/// Zero is admitted so that the least failure below is the least positive non-divisor. Zero
/// divides only zero, so without this the search would stop at zero for every other `n` and
/// say nothing.
define divides_or_zero(n: Nat, m: Nat) -> Bool {
    m = Nat.0 or m.divides(n)
}

/// The condition, spelled out.
theorem divides_or_zero_eq(n: Nat, m: Nat) {
    divides_or_zero(n)(m) = (m = Nat.0 or m.divides(n))
}

/// True if `m` is the smallest positive natural that does not divide `n`.
define is_least_non_divisor(n: Nat, m: Nat) -> Bool {
    is_least_failure(divides_or_zero(n), m)
}

/// The least non-divisor does not divide.
theorem least_non_divisor_not_divides(n: Nat, m: Nat) {
    is_least_non_divisor(n, m) implies not m.divides(n)
} by {
    if is_least_non_divisor(n, m) {
        is_least_non_divisor(n, m) = is_least_failure(divides_or_zero(n), m)
        is_least_failure(divides_or_zero(n), m)
        is_least_failure_fails(divides_or_zero(n), m)
        not divides_or_zero(n)(m)
        divides_or_zero_eq(n, m)
        not (m = Nat.0 or m.divides(n))
        not m.divides(n)
    }
}

/// Every positive natural below the least non-divisor divides.
theorem least_non_divisor_divides_below(n: Nat, m: Nat, k: Nat) {
    is_least_non_divisor(n, m) and k < m and k != Nat.0 implies k.divides(n)
} by {
    if is_least_non_divisor(n, m) and k < m and k != Nat.0 {
        is_least_non_divisor(n, m) = is_least_failure(divides_or_zero(n), m)
        is_least_failure(divides_or_zero(n), m)
        is_least_failure_holds_below(divides_or_zero(n), m, k)
        divides_or_zero(n)(k)
        divides_or_zero_eq(n, k)
        (k = Nat.0 or k.divides(n))
        k.divides(n)
    }
}

/// Every nonzero natural has a least non-divisor.
///
/// One more than the number itself cannot divide it, since a divisor of a nonzero natural is
/// at most that natural.
theorem least_non_divisor_exists(n: Nat) {
    n != Nat.0 implies exists(m: Nat) { is_least_non_divisor(n, m) }
} by {
    if n != Nat.0 {
        if (n + Nat.1).divides(n) {
            divides_lte(n + Nat.1, n)
            (n = Nat.0 or n + Nat.1 <= n)
            n + Nat.1 <= n
            n < n + Nat.1
            false
        }
        not (n + Nat.1).divides(n)
        n + Nat.1 != Nat.0
        divides_or_zero_eq(n, n + Nat.1)
        not divides_or_zero(n)(n + Nat.1)
        least_failure_exists(divides_or_zero(n), n + Nat.1)
        exists(k: Nat) {
            is_least_failure(divides_or_zero(n), k)
        }
        let (m: Nat) satisfy {
            is_least_failure(divides_or_zero(n), m)
        }
        is_least_non_divisor(n, m) = is_least_failure(divides_or_zero(n), m)
        is_least_non_divisor(n, m)
        exists(k: Nat) { is_least_non_divisor(n, k) }
    }
}

/// The least non-divisor is at least two.
///
/// Zero passes by convention and one divides everything, so neither can be the least failure.
theorem least_non_divisor_at_least_two(n: Nat, m: Nat) {
    is_least_non_divisor(n, m) implies Nat.2 <= m
} by {
    if is_least_non_divisor(n, m) {
        is_least_non_divisor(n, m) = is_least_failure(divides_or_zero(n), m)
        is_least_failure(divides_or_zero(n), m)
        forall(x: Nat) {
            if x < Nat.2 {
                if x = Nat.0 {
                    divides_or_zero_eq(n, x)
                    divides_or_zero(n)(x)
                }
                if x != Nat.0 {
                    lt_imp_lte_suc(x, Nat.2)
                    Nat.2 = Nat.1.suc
                    x.suc <= Nat.1.suc
                    lte_cancel_suc(x, Nat.1)
                    x <= Nat.1
                    Nat.0 < x
                    lt_imp_lte_suc(Nat.0, x)
                    Nat.0.suc <= x
                    Nat.1 <= x
                    lte_antisymm(x, Nat.1)
                    x = Nat.1
                    Nat.1.divides(n)
                    x.divides(n)
                    divides_or_zero_eq(n, x)
                    divides_or_zero(n)(x)
                }
                divides_or_zero(n)(x)
            }
            (x < Nat.2 implies divides_or_zero(n)(x))
        }
        least_failure_lower_bound(divides_or_zero(n), m, Nat.2)
        Nat.2 <= m
    }
}

/// The least non-divisor is unique.
theorem least_non_divisor_unique(n: Nat, m: Nat, m2: Nat) {
    is_least_non_divisor(n, m) and is_least_non_divisor(n, m2) implies m = m2
} by {
    if is_least_non_divisor(n, m) and is_least_non_divisor(n, m2) {
        is_least_non_divisor(n, m) = is_least_failure(divides_or_zero(n), m)
        is_least_non_divisor(n, m2) = is_least_failure(divides_or_zero(n), m2)
        least_failure_unique(divides_or_zero(n), m, m2)
        m = m2
    }
}

/// A factor of a product by something above one is strictly smaller than the product.
theorem lt_mul_of_one_lt_right(a: Nat, b: Nat) {
    Nat.0 < a and Nat.1 < b implies a < a * b
} by {
    if Nat.0 < a and Nat.1 < b {
        Nat.2 <= b
        lte_mul_both(a, Nat.2, b)
        a * Nat.2 <= a * b
        a * Nat.2 = a + a
        a < a + a
        lt_and_lte(a, a + a, a * b)
        a < a * b
    }
}

/// Coprime divisors multiply to a divisor.
///
/// Their least common multiple divides, and coprimality makes that the product.
theorem coprime_divisors_product_divides(a: Nat, b: Nat, n: Nat) {
    a.divides(n) and b.divides(n) and a.coprime(b) implies (a * b).divides(n)
} by {
    if a.divides(n) and b.divides(n) and a.coprime(b) {
        lcm_divides_of_common(a, b, n)
        a.lcm(b).divides(n)
        a.coprime(b) = (a.gcd(b) = Nat.1)
        a.gcd(b) = Nat.1
        gcd_mul_lcm(a, b)
        a.gcd(b) * a.lcm(b) = a * b
        Nat.1 * a.lcm(b) = a * b
        a.lcm(b) = a * b
        (a * b).divides(n)
    }
}

/// The least non-divisor does not split into coprime factors above one.
///
/// Both factors would be smaller, hence divisors, and coprime divisors multiply to a divisor,
/// which the least non-divisor is not. This is the whole content of the least non-divisor
/// being a prime power: a number with two distinct prime factors splits exactly this way.
theorem least_non_divisor_factors_not_coprime(n: Nat, m: Nat, a: Nat, b: Nat) {
    is_least_non_divisor(n, m) and m = a * b and Nat.1 < a and Nat.1 < b
        implies not a.coprime(b)
} by {
    if is_least_non_divisor(n, m) and m = a * b and Nat.1 < a and Nat.1 < b {
        Nat.0 < Nat.1
        lt_trans(Nat.0, Nat.1, a)
        Nat.0 < a
        lt_mul_of_one_lt_right(a, b)
        a < a * b
        a < m
        a != Nat.0
        least_non_divisor_divides_below(n, m, a)
        a.divides(n)
        lt_trans(Nat.0, Nat.1, b)
        Nat.0 < b
        lt_mul_of_one_lt_right(b, a)
        b < b * a
        b * a = a * b
        b < m
        b != Nat.0
        least_non_divisor_divides_below(n, m, b)
        b.divides(n)
        if a.coprime(b) {
            coprime_divisors_product_divides(a, b, n)
            (a * b).divides(n)
            m.divides(n)
            least_non_divisor_not_divides(n, m)
            not m.divides(n)
            false
        }
        not a.coprime(b)
    }
}

/// A proper divisor of the least non-divisor divides the number.
theorem least_non_divisor_proper_divisor_divides(n: Nat, m: Nat, d: Nat) {
    is_least_non_divisor(n, m) and d.divides(m) and d != m and d != Nat.0
        implies d.divides(n)
} by {
    if is_least_non_divisor(n, m) and d.divides(m) and d != m and d != Nat.0 {
        divides_lte(d, m)
        (m = Nat.0 or d <= m)
        if m = Nat.0 {
            least_non_divisor_at_least_two(n, m)
            Nat.2 <= m
            Nat.2 <= Nat.0
            only_zero_lte_zero(Nat.2)
            Nat.2 = Nat.0
            false
        }
        m != Nat.0
        d <= m
        d < m
        least_non_divisor_divides_below(n, m, d)
        d.divides(n)
    }
}
