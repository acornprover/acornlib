from nat import Nat, small_mod
from int import Int
from list import List
from pair import Pair
from zmod import int_mod_rel
from number_theory import congruence_class_contains, covers_int, covers_int_nil_false,
    covers_int_cons_imp, nat_congr_mod_iff_int_mod_rel
from data.nat.nat_residue_count import in_residue_class, in_residue_class_apply
from data.nat.nat_density_subadditive import or_pred, or_pred_apply
from real import Real, lte_trans
from nat import from_nat
from data.nat.nat_counting_function import count_upto, count_upto_zero, count_upto_suc_true
from data.nat.nat_density import range_size, range_size_positive, density_seq
from analysis import is_bounded_above, is_bounded_below, liminf, tail_inf,
    tail_inf_is_greatest, liminf_ge_tail_inf
from data.nat.nat_density_extremes import upper_density, lower_density, density_seq_bounded_above,
    density_seq_bounded_below, lower_density_le_upper_density
from data.nat.nat_covering_density import covers_nat, covers_nat_cons, covers_nat_nil,
    system_reduced, system_reduced_head, system_reduced_tail, system_density_real,
    upper_density_covers_nat_le

numerals Nat

/// A natural in a congruence class lies in the matching residue class.
///
/// The class condition is stated over the integers and the residue class over the naturals; the
/// bridge between the two congruences carries one to the other, and the residue being below the
/// modulus is what makes the two remainders agree on the nose.
theorem congruence_class_contains_imp_in_residue_class(cl: Pair[Nat, Nat], k: Nat) {
    cl.second < cl.first and congruence_class_contains(cl, Int.from_nat(k))
        implies in_residue_class(cl.first, cl.second)(k)
} by {
    if cl.second < cl.first and congruence_class_contains(cl, Int.from_nat(k)) {
        (congruence_class_contains(cl, Int.from_nat(k))
            = int_mod_rel(cl.first, Int.from_nat(k), Int.from_nat(cl.second)))
        int_mod_rel(cl.first, Int.from_nat(k), Int.from_nat(cl.second))
        nat_congr_mod_iff_int_mod_rel(k, cl.second, cl.first)
        (k.congr_mod(cl.second, cl.first)
            = int_mod_rel(cl.first, Int.from_nat(k), Int.from_nat(cl.second)))
        k.congr_mod(cl.second, cl.first)
        (k.congr_mod(cl.second, cl.first)
            = (k.mod(cl.first) = cl.second.mod(cl.first)))
        k.mod(cl.first) = cl.second.mod(cl.first)
        small_mod(cl.second, cl.first)
        cl.second.mod(cl.first) = cl.second
        k.mod(cl.first) = cl.second
        in_residue_class_apply(cl.first, cl.second, k)
        (in_residue_class(cl.first, cl.second)(k) = (k.mod(cl.first) = cl.second))
        in_residue_class(cl.first, cl.second)(k)
    }
}

/// A system covering an integer covers the corresponding natural.
///
/// Induction along the list, class by class.
theorem covers_int_imp_covers_nat(system: List[Pair[Nat, Nat]], k: Nat) {
    system_reduced(system) and covers_int(system, Int.from_nat(k))
        implies covers_nat(system)(k)
} by {
    define p(s: List[Pair[Nat, Nat]]) -> Bool {
        system_reduced(s) and covers_int(s, Int.from_nat(k)) implies covers_nat(s)(k)
    }
    if covers_int(List.nil[Pair[Nat, Nat]], Int.from_nat(k)) {
        covers_int_nil_false(Int.from_nat(k))
        not covers_int(List.nil[Pair[Nat, Nat]], Int.from_nat(k))
        false
    }
    not covers_int(List.nil[Pair[Nat, Nat]], Int.from_nat(k))
    p(List.nil[Pair[Nat, Nat]])
    forall(head: Pair[Nat, Nat], tail: List[Pair[Nat, Nat]]) {
        if p(tail) {
            if system_reduced(List.cons(head, tail))
                and covers_int(List.cons(head, tail), Int.from_nat(k)) {
                system_reduced_head(head, tail)
                head.second < head.first
                system_reduced_tail(head, tail)
                system_reduced(tail)
                covers_int_cons_imp(head, tail, Int.from_nat(k))
                (congruence_class_contains(head, Int.from_nat(k))
                    or covers_int(tail, Int.from_nat(k)))
                covers_nat_cons(head, tail)
                (covers_nat(List.cons(head, tail))
                    = or_pred(in_residue_class(head.first, head.second), covers_nat(tail)))
                or_pred_apply(in_residue_class(head.first, head.second), covers_nat(tail), k)
                (or_pred(in_residue_class(head.first, head.second), covers_nat(tail))(k)
                    = (in_residue_class(head.first, head.second)(k) or covers_nat(tail)(k)))
                if congruence_class_contains(head, Int.from_nat(k)) {
                    congruence_class_contains_imp_in_residue_class(head, k)
                    in_residue_class(head.first, head.second)(k)
                    (in_residue_class(head.first, head.second)(k) or covers_nat(tail)(k))
                }
                if not congruence_class_contains(head, Int.from_nat(k)) {
                    covers_int(tail, Int.from_nat(k))
                    (system_reduced(tail) and covers_int(tail, Int.from_nat(k))
                        implies covers_nat(tail)(k))
                    covers_nat(tail)(k)
                    (in_residue_class(head.first, head.second)(k) or covers_nat(tail)(k))
                }
                (in_residue_class(head.first, head.second)(k) or covers_nat(tail)(k))
                or_pred(in_residue_class(head.first, head.second), covers_nat(tail))(k)
                covers_nat(List.cons(head, tail))(k)
            }
            p(List.cons(head, tail))
        }
        (p(tail) implies p(List.cons(head, tail)))
    }
    p(List.nil[Pair[Nat, Nat]]) and forall(h: Pair[Nat, Nat], t: List[Pair[Nat, Nat]]) {
        p(t) implies p(List.cons(h, t))
    }
    List.induction(p)
    p(system)
}

/// True if every integer lies in some class of the system.
define is_covering_system(system: List[Pair[Nat, Nat]]) -> Bool {
    forall(x: Int) {
        covers_int(system, x)
    }
}

/// A covering system covers every natural.
theorem covering_system_covers_nat(system: List[Pair[Nat, Nat]], k: Nat) {
    system_reduced(system) and is_covering_system(system)
        implies covers_nat(system)(k)
} by {
    if system_reduced(system) and is_covering_system(system) {
        (is_covering_system(system) = forall(x: Int) { covers_int(system, x) })
        covers_int(system, Int.from_nat(k))
        covers_int_imp_covers_nat(system, k)
        covers_nat(system)(k)
    }
}

/// A predicate holding everywhere is counted at every step.
theorem count_upto_of_all(q: Nat -> Bool, n: Nat) {
    (forall(i: Nat) { q(i) }) implies count_upto(q, n) = n
} by {
    if forall(i: Nat) { q(i) } {
        define w(x: Nat) -> Bool {
            count_upto(q, x) = x
        }
        count_upto_zero(q)
        w(Nat.0)
        forall(j: Nat) {
            if w(j) {
                count_upto(q, j) = j
                q(j)
                count_upto_suc_true(q, j)
                count_upto(q, j.suc) = count_upto(q, j) + Nat.1
                (j + Nat.1 = j.suc)
                count_upto(q, j.suc) = j.suc
                w(j.suc)
            }
            (w(j) implies w(j.suc))
        }
        w(Nat.0) and forall(j: Nat) {
            w(j) implies w(j.suc)
        }
        Nat.induction(w)
        w(n)
        count_upto(q, n) = n
    }
}

/// The reciprocals of the moduli of a covering system sum to at least one.
///
/// The classes cover every natural, so the fraction covered is one at every range, hence the
/// tail infimum is one and so is the lower density. The density of what a system covers is at
/// most the sum of the reciprocals of its moduli.
///
/// The bound goes through the tail infimum rather than by citing the density-one theorem,
/// because that citation will not instantiate at the partially applied `covers_nat(system)`.
theorem covering_system_density_ge_one(system: List[Pair[Nat, Nat]]) {
    system_reduced(system) and is_covering_system(system)
        implies Real.1 <= system_density_real(system)
} by {
    if system_reduced(system) and is_covering_system(system) {
        forall(m: Nat) {
            covering_system_covers_nat(system, m)
            covers_nat(system)(m)
        }
        density_seq_bounded_above(covers_nat(system))
        is_bounded_above(density_seq(covers_nat(system)))
        density_seq_bounded_below(covers_nat(system))
        is_bounded_below(density_seq(covers_nat(system)))
        forall(n: Nat) {
            count_upto_of_all(covers_nat(system), n.suc)
            count_upto(covers_nat(system), n.suc) = n.suc
            (range_size(n) = from_nat[Real](n.suc))
            (density_seq(covers_nat(system), n)
                = from_nat[Real](count_upto(covers_nat(system), n.suc)) / range_size(n))
            (density_seq(covers_nat(system), n) = range_size(n) / range_size(n))
            range_size_positive(n)
            Real.0 < range_size(n)
            range_size(n) != Real.0
            (range_size(n) / range_size(n) = Real.1)
            density_seq(covers_nat(system), n) = Real.1
            (density_seq(covers_nat(system))(n) = density_seq(covers_nat(system), n))
            Real.1 <= density_seq(covers_nat(system))(n)
            (Nat.0 <= n implies Real.1 <= density_seq(covers_nat(system))(n))
        }
        tail_inf_is_greatest(density_seq(covers_nat(system)), Nat.0, Real.1)
        Real.1 <= tail_inf(density_seq(covers_nat(system)), Nat.0)
        liminf_ge_tail_inf(density_seq(covers_nat(system)), Nat.0)
        (tail_inf(density_seq(covers_nat(system)), Nat.0)
            <= liminf(density_seq(covers_nat(system))))
        lte_trans(Real.1, tail_inf(density_seq(covers_nat(system)), Nat.0),
            liminf(density_seq(covers_nat(system))))
        Real.1 <= liminf(density_seq(covers_nat(system)))
        (lower_density(covers_nat(system)) = liminf(density_seq(covers_nat(system))))
        Real.1 <= lower_density(covers_nat(system))
        lower_density_le_upper_density(covers_nat(system))
        (lower_density(covers_nat(system)) <= upper_density(covers_nat(system)))
        lte_trans(Real.1, lower_density(covers_nat(system)),
            upper_density(covers_nat(system)))
        Real.1 <= upper_density(covers_nat(system))
        upper_density_covers_nat_le(system)
        upper_density(covers_nat(system)) <= system_density_real(system)
        lte_trans(Real.1, upper_density(covers_nat(system)), system_density_real(system))
        Real.1 <= system_density_real(system)
    }
}
