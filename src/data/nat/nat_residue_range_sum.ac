from nat import Nat, mod_lt, lt_and_lte
from algebra.add_comm_monoid import AddCommMonoid
from data.basic.functions import function_extensionality
from data.nat.nat_range_sum import range_sum, range_sum_zero, range_sum_suc, range_sum_congr,
    range_sum_add, add_fn, shift_fn, range_sum_split_twice, zero_fn, range_sum_zero_fn

numerals Nat

/// The summand supported at a single index.
///
/// The indicator of one point, weighted. Written as a conditional here and nowhere else: the two
/// case lemmas below are what every statement downstream uses.
define point_fn[A: AddCommMonoid](p: Nat, v: A, a: Nat) -> A {
    if a = p {
        v
    } else {
        A.0
    }
}

/// At its own index the point summand is its value.
theorem point_fn_at[A: AddCommMonoid](p: Nat, v: A) {
    point_fn(p, v, p) = v
}

/// Away from its index the point summand vanishes.
theorem point_fn_off[A: AddCommMonoid](p: Nat, v: A, a: Nat) {
    a != p implies point_fn(p, v, a) = A.0
}

/// A summand vanishing below the limit sums to zero.
theorem range_sum_zero_of_vanishes[A: AddCommMonoid](f: Nat -> A, n: Nat) {
    (forall(i: Nat) { i < n implies f(i) = A.0 }) implies range_sum(f, n) = A.0
} by {
    if forall(i: Nat) { i < n implies f(i) = A.0 } {
        forall(i: Nat) {
            if i < n {
                f(i) = A.0
                (zero_fn[A](i) = A.0)
                f(i) = zero_fn[A](i)
            }
            (i < n implies f(i) = zero_fn[A](i))
        }
        range_sum_congr(f, zero_fn[A], n)
        range_sum(f, n) = range_sum(zero_fn[A], n)
        range_sum_zero_fn[A](n)
        range_sum(zero_fn[A], n) = A.0
        range_sum(f, n) = A.0
    }
}

/// A point summand inside the range sums to its value.
///
/// The range splits at the point and just past it. The two outer parts vanish termwise and the
/// middle part is a single term, so nothing has to be counted.
theorem range_sum_point_fn[A: AddCommMonoid](p: Nat, v: A, m: Nat) {
    p < m implies range_sum(point_fn(p, v), m) = v
} by {
    if p < m {
        p.suc <= m
        let (c: Nat) satisfy {
            p.suc + c = m
        }
        (p + (Nat.1 + c) = m)
        range_sum_split_twice(point_fn(p, v), p, Nat.1, c)
        (range_sum(point_fn(p, v), p + (Nat.1 + c))
            = (range_sum(point_fn(p, v), p) + range_sum(shift_fn(point_fn(p, v), p), Nat.1))
                + range_sum(shift_fn(shift_fn(point_fn(p, v), p), Nat.1), c))
        forall(i: Nat) {
            if i < p {
                i != p
                point_fn_off(p, v, i)
                point_fn(p, v, i) = A.0
            }
            (i < p implies point_fn(p, v)(i) = A.0)
        }
        range_sum_zero_of_vanishes(point_fn(p, v), p)
        range_sum(point_fn(p, v), p) = A.0
        (shift_fn(point_fn(p, v), p, Nat.0) = point_fn(p, v, p + Nat.0))
        (p + Nat.0 = p)
        point_fn_at(p, v)
        (shift_fn(point_fn(p, v), p)(Nat.0) = v)
        range_sum_suc(shift_fn(point_fn(p, v), p), Nat.0)
        (range_sum(shift_fn(point_fn(p, v), p), Nat.0.suc)
            = range_sum(shift_fn(point_fn(p, v), p), Nat.0)
                + shift_fn(point_fn(p, v), p)(Nat.0))
        range_sum_zero(shift_fn(point_fn(p, v), p))
        (range_sum(shift_fn(point_fn(p, v), p), Nat.0) = A.0)
        (A.0 + v = v)
        (Nat.0.suc = Nat.1)
        range_sum(shift_fn(point_fn(p, v), p), Nat.1) = v
        forall(i: Nat) {
            (shift_fn(shift_fn(point_fn(p, v), p), Nat.1, i)
                = shift_fn(point_fn(p, v), p)(Nat.1 + i))
            (shift_fn(point_fn(p, v), p, Nat.1 + i) = point_fn(p, v, p + (Nat.1 + i)))
            (p + (Nat.1 + i) = p.suc + i)
            p < p.suc
            p.suc <= p.suc + i
            lt_and_lte(p, p.suc, p.suc + i)
            p < p.suc + i
            p.suc + i != p
            point_fn_off(p, v, p.suc + i)
            (point_fn(p, v, p.suc + i) = A.0)
            (i < c implies shift_fn(shift_fn(point_fn(p, v), p), Nat.1)(i) = A.0)
        }
        range_sum_zero_of_vanishes(shift_fn(shift_fn(point_fn(p, v), p), Nat.1), c)
        (range_sum(shift_fn(shift_fn(point_fn(p, v), p), Nat.1), c) = A.0)
        ((A.0 + v) + A.0 = v)
        range_sum(point_fn(p, v), m) = v
    }
}

/// The summand of `f` masked to one residue class modulo `m`.
define residue_summand[A: AddCommMonoid](m: Nat, a: Nat, f: Nat -> A, i: Nat) -> A {
    if i.mod(m) = a {
        f(i)
    } else {
        A.0
    }
}

/// Inside the class the masked summand is the original.
theorem residue_summand_in[A: AddCommMonoid](m: Nat, a: Nat, f: Nat -> A, i: Nat) {
    i.mod(m) = a implies residue_summand(m, a, f, i) = f(i)
}

/// Outside the class the masked summand vanishes.
theorem residue_summand_out[A: AddCommMonoid](m: Nat, a: Nat, f: Nat -> A, i: Nat) {
    i.mod(m) != a implies residue_summand(m, a, f, i) = A.0
}

/// The partial sum of `f` over one residue class below `n`.
define residue_range_sum[A: AddCommMonoid](m: Nat, a: Nat, f: Nat -> A, n: Nat) -> A {
    range_sum(residue_summand(m, a, f), n)
}

/// The empty range contributes nothing to any class.
theorem residue_range_sum_zero[A: AddCommMonoid](m: Nat, a: Nat, f: Nat -> A) {
    residue_range_sum(m, a, f, Nat.0) = A.0
} by {
    (residue_range_sum(m, a, f, Nat.0) = range_sum(residue_summand(m, a, f), Nat.0))
    range_sum_zero(residue_summand(m, a, f))
    residue_range_sum(m, a, f, Nat.0) = A.0
}

/// One more index adds its masked term.
theorem residue_range_sum_suc[A: AddCommMonoid](m: Nat, a: Nat, f: Nat -> A, n: Nat) {
    residue_range_sum(m, a, f, n.suc)
        = residue_range_sum(m, a, f, n) + residue_summand(m, a, f, n)
} by {
    (residue_range_sum(m, a, f, n.suc) = range_sum(residue_summand(m, a, f), n.suc))
    range_sum_suc(residue_summand(m, a, f), n)
    (range_sum(residue_summand(m, a, f), n.suc)
        = range_sum(residue_summand(m, a, f), n) + residue_summand(m, a, f)(n))
    (residue_range_sum(m, a, f, n) = range_sum(residue_summand(m, a, f), n))
    (residue_range_sum(m, a, f, n.suc)
        = residue_range_sum(m, a, f, n) + residue_summand(m, a, f, n))
}

/// The residue-class partial sums, indexed by the residue.
///
/// Packaged as a function of the residue so that summing over the classes is a range sum and
/// needs no inline lambda.
define residue_class_sums[A: AddCommMonoid](m: Nat, f: Nat -> A, n: Nat, a: Nat) -> A {
    residue_range_sum(m, a, f, n)
}

/// The increment each class receives when the range grows by one, indexed by the residue.
define residue_class_delta[A: AddCommMonoid](m: Nat, f: Nat -> A, n: Nat, a: Nat) -> A {
    residue_summand(m, a, f, n)
}

/// Only the class of the new index receives anything.
///
/// The increment is the point summand at the residue of the new index, which is why summing the
/// increments over the classes returns the single term.
theorem residue_class_delta_eq_point[A: AddCommMonoid](m: Nat, f: Nat -> A, n: Nat) {
    residue_class_delta(m, f, n) = point_fn(n.mod(m), f(n))
} by {
    forall(a: Nat) {
        (residue_class_delta(m, f, n)(a) = residue_summand(m, a, f, n))
        if a = n.mod(m) {
            (n.mod(m) = a)
            residue_summand_in(m, a, f, n)
            (residue_summand(m, a, f, n) = f(n))
            point_fn_at(n.mod(m), f(n))
            (point_fn(n.mod(m), f(n))(a) = f(n))
            residue_class_delta(m, f, n)(a) = point_fn(n.mod(m), f(n))(a)
        }
        if a != n.mod(m) {
            (n.mod(m) != a)
            residue_summand_out(m, a, f, n)
            (residue_summand(m, a, f, n) = A.0)
            point_fn_off(n.mod(m), f(n), a)
            (point_fn(n.mod(m), f(n))(a) = A.0)
            residue_class_delta(m, f, n)(a) = point_fn(n.mod(m), f(n))(a)
        }
        residue_class_delta(m, f, n)(a) = point_fn(n.mod(m), f(n))(a)
    }
    function_extensionality[Nat, A](residue_class_delta(m, f, n), point_fn(n.mod(m), f(n)))
    residue_class_delta(m, f, n) = point_fn(n.mod(m), f(n))
}

/// Growing the range by one adds the increment to every class at once.
theorem residue_class_sums_suc[A: AddCommMonoid](m: Nat, f: Nat -> A, n: Nat) {
    residue_class_sums(m, f, n.suc)
        = add_fn(residue_class_sums(m, f, n), residue_class_delta(m, f, n))
} by {
    forall(a: Nat) {
        (residue_class_sums(m, f, n.suc)(a) = residue_range_sum(m, a, f, n.suc))
        residue_range_sum_suc(m, a, f, n)
        (residue_range_sum(m, a, f, n.suc)
            = residue_range_sum(m, a, f, n) + residue_summand(m, a, f, n))
        (residue_class_sums(m, f, n)(a) = residue_range_sum(m, a, f, n))
        (residue_class_delta(m, f, n)(a) = residue_summand(m, a, f, n))
        (add_fn(residue_class_sums(m, f, n), residue_class_delta(m, f, n))(a)
            = residue_class_sums(m, f, n)(a) + residue_class_delta(m, f, n)(a))
        (residue_class_sums(m, f, n.suc)(a)
            = add_fn(residue_class_sums(m, f, n), residue_class_delta(m, f, n))(a))
    }
    function_extensionality[Nat, A](residue_class_sums(m, f, n.suc),
        add_fn(residue_class_sums(m, f, n), residue_class_delta(m, f, n)))
    (residue_class_sums(m, f, n.suc)
        = add_fn(residue_class_sums(m, f, n), residue_class_delta(m, f, n)))
}

/// A range sum is the sum of its residue-class partial sums.
///
/// The decomposition that makes a residue-class condition usable inside a sum: every index below
/// the limit lies in exactly one class modulo `m`, so summing the classes recovers the whole.
/// Induction on the limit, where the new index lands in a single class and the increments over
/// the classes collapse to one term.
theorem range_sum_eq_residue_class_sums[A: AddCommMonoid](m: Nat, f: Nat -> A, n: Nat) {
    m != Nat.0 implies range_sum(f, n) = range_sum(residue_class_sums(m, f, n), m)
} by {
    if m != Nat.0 {
        define w(k: Nat) -> Bool {
            range_sum(f, k) = range_sum(residue_class_sums(m, f, k), m)
        }
        range_sum_zero(f)
        (range_sum(f, Nat.0) = A.0)
        forall(a: Nat) {
            residue_range_sum_zero(m, a, f)
            (residue_class_sums(m, f, Nat.0)(a) = A.0)
            (a < m implies residue_class_sums(m, f, Nat.0)(a) = A.0)
        }
        range_sum_zero_of_vanishes(residue_class_sums(m, f, Nat.0), m)
        (range_sum(residue_class_sums(m, f, Nat.0), m) = A.0)
        w(Nat.0)
        forall(k: Nat) {
            if w(k) {
                (range_sum(f, k) = range_sum(residue_class_sums(m, f, k), m))
                residue_class_sums_suc(m, f, k)
                (residue_class_sums(m, f, k.suc)
                    = add_fn(residue_class_sums(m, f, k), residue_class_delta(m, f, k)))
                range_sum_add(residue_class_sums(m, f, k), residue_class_delta(m, f, k), m)
                (range_sum(add_fn(residue_class_sums(m, f, k),
                    residue_class_delta(m, f, k)), m)
                    = range_sum(residue_class_sums(m, f, k), m)
                        + range_sum(residue_class_delta(m, f, k), m))
                residue_class_delta_eq_point(m, f, k)
                (residue_class_delta(m, f, k) = point_fn(k.mod(m), f(k)))
                mod_lt(k, m)
                k.mod(m) < m
                range_sum_point_fn(k.mod(m), f(k), m)
                (range_sum(point_fn(k.mod(m), f(k)), m) = f(k))
                (range_sum(residue_class_delta(m, f, k), m) = f(k))
                (range_sum(residue_class_sums(m, f, k.suc), m)
                    = range_sum(residue_class_sums(m, f, k), m) + f(k))
                range_sum_suc(f, k)
                (range_sum(f, k.suc) = range_sum(f, k) + f(k))
                range_sum(f, k.suc) = range_sum(residue_class_sums(m, f, k.suc), m)
                w(k.suc)
            }
            (w(k) implies w(k.suc))
        }
        w(Nat.0) and forall(k: Nat) {
            w(k) implies w(k.suc)
        }
        Nat.induction(w)
        w(n)
        range_sum(f, n) = range_sum(residue_class_sums(m, f, n), m)
    }
}
