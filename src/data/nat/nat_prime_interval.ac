from nat import Nat, lte_and_lt, lte_trans, divides_trans
from data.nat.nat_smooth import is_smooth, is_rough, is_smooth_apply, is_rough_apply,
    is_smooth_intro, is_rough_intro

numerals Nat

/// True if some prime greater than `lo` and at most `hi` divides `n`.
///
/// The half-open interval matches the conventions of smoothness and roughness, so that
/// `is_rough(n, lo)` and `is_smooth(n, hi)` together say every prime factor lies here.
define has_prime_factor_in(n: Nat, lo: Nat, hi: Nat) -> Bool {
    exists(p: Nat) {
        p.is_prime and lo < p and p <= hi and p.divides(n)
    }
}

/// A prime in the interval dividing `n` witnesses the predicate.
theorem has_prime_factor_in_intro(n: Nat, lo: Nat, hi: Nat, p: Nat) {
    p.is_prime and lo < p and p <= hi and p.divides(n) implies has_prime_factor_in(n, lo, hi)
} by {
    if p.is_prime and lo < p and p <= hi and p.divides(n) {
        has_prime_factor_in(n, lo, hi) = exists(q: Nat) {
            q.is_prime and lo < q and q <= hi and q.divides(n)
        }
        exists(q: Nat) {
            q.is_prime and lo < q and q <= hi and q.divides(n)
        }
        has_prime_factor_in(n, lo, hi)
    }
}

/// The predicate yields such a prime.
theorem has_prime_factor_in_witness(n: Nat, lo: Nat, hi: Nat) {
    has_prime_factor_in(n, lo, hi) implies exists(p: Nat) {
        p.is_prime and lo < p and p <= hi and p.divides(n)
    }
} by {
    if has_prime_factor_in(n, lo, hi) {
        has_prime_factor_in(n, lo, hi) = exists(q: Nat) {
            q.is_prime and lo < q and q <= hi and q.divides(n)
        }
        exists(q: Nat) {
            q.is_prime and lo < q and q <= hi and q.divides(n)
        }
    }
}

/// Widening the interval preserves the predicate.
theorem has_prime_factor_in_widen(n: Nat, lo: Nat, hi: Nat, lo2: Nat, hi2: Nat) {
    has_prime_factor_in(n, lo, hi) and lo2 <= lo and hi <= hi2
        implies has_prime_factor_in(n, lo2, hi2)
} by {
    if has_prime_factor_in(n, lo, hi) and lo2 <= lo and hi <= hi2 {
        has_prime_factor_in_witness(n, lo, hi)
        let (p: Nat) satisfy {
            p.is_prime and lo < p and p <= hi and p.divides(n)
        }
        lte_and_lt(lo2, lo, p)
        lo2 < p
        lte_trans(p, hi, hi2)
        p <= hi2
        has_prime_factor_in_intro(n, lo2, hi2, p)
        has_prime_factor_in(n, lo2, hi2)
    }
}

/// A prime factor of a divisor is a prime factor of the original in the same interval.
theorem has_prime_factor_in_of_divides(n: Nat, m: Nat, lo: Nat, hi: Nat) {
    has_prime_factor_in(m, lo, hi) and m.divides(n) implies has_prime_factor_in(n, lo, hi)
} by {
    if has_prime_factor_in(m, lo, hi) and m.divides(n) {
        has_prime_factor_in_witness(m, lo, hi)
        let (p: Nat) satisfy {
            p.is_prime and lo < p and p <= hi and p.divides(m)
        }
        divides_trans(p, m, n)
        p.divides(n)
        has_prime_factor_in_intro(n, lo, hi, p)
        has_prime_factor_in(n, lo, hi)
    }
}

/// True if every prime factor of `n` is greater than `lo` and at most `hi`.
///
/// The conjunction of roughness at the lower end and smoothness at the upper end.
define all_prime_factors_in(n: Nat, lo: Nat, hi: Nat) -> Bool {
    is_rough(n, lo) and is_smooth(n, hi)
}

/// A prime factor of such a number lies in the interval.
theorem all_prime_factors_in_apply(n: Nat, lo: Nat, hi: Nat, p: Nat) {
    all_prime_factors_in(n, lo, hi) and p.is_prime and p.divides(n)
        implies lo < p and p <= hi
} by {
    if all_prime_factors_in(n, lo, hi) and p.is_prime and p.divides(n) {
        all_prime_factors_in(n, lo, hi) = (is_rough(n, lo) and is_smooth(n, hi))
        is_rough(n, lo)
        is_smooth(n, hi)
        is_rough_apply(n, lo, p)
        lo < p
        is_smooth_apply(n, hi, p)
        p <= hi
        lo < p and p <= hi
    }
}

/// A pointwise interval bound on prime factors gives the predicate.
theorem all_prime_factors_in_intro(n: Nat, lo: Nat, hi: Nat) {
    (forall(p: Nat) { p.is_prime and p.divides(n) implies lo < p and p <= hi })
        implies all_prime_factors_in(n, lo, hi)
} by {
    if forall(p: Nat) { p.is_prime and p.divides(n) implies lo < p and p <= hi } {
        forall(p: Nat) {
            if p.is_prime and p.divides(n) {
                lo < p and p <= hi
                lo < p
            }
            p.is_prime and p.divides(n) implies lo < p
        }
        is_rough_intro(n, lo)
        is_rough(n, lo)
        forall(p: Nat) {
            if p.is_prime and p.divides(n) {
                lo < p and p <= hi
                p <= hi
            }
            p.is_prime and p.divides(n) implies p <= hi
        }
        is_smooth_intro(n, hi)
        is_smooth(n, hi)
        all_prime_factors_in(n, lo, hi) = (is_rough(n, lo) and is_smooth(n, hi))
        all_prime_factors_in(n, lo, hi)
    }
}

/// A prime factor found in a wider range in fact lies in the interval.
///
/// The two predicates fit together: the existential locates a prime factor loosely, and the
/// universal one pins it down to the interval.
theorem prime_factor_lands_in_interval(n: Nat, lo: Nat, hi: Nat, lo2: Nat, hi2: Nat) {
    all_prime_factors_in(n, lo, hi) and has_prime_factor_in(n, lo2, hi2)
        implies has_prime_factor_in(n, lo, hi)
} by {
    if all_prime_factors_in(n, lo, hi) and has_prime_factor_in(n, lo2, hi2) {
        has_prime_factor_in_witness(n, lo2, hi2)
        let (p: Nat) satisfy {
            p.is_prime and lo2 < p and p <= hi2 and p.divides(n)
        }
        all_prime_factors_in_apply(n, lo, hi, p)
        lo < p
        p <= hi
        has_prime_factor_in_intro(n, lo, hi, p)
        has_prime_factor_in(n, lo, hi)
    }
}

/// The interval condition passes to divisors.
theorem all_prime_factors_in_of_divides(n: Nat, m: Nat, lo: Nat, hi: Nat) {
    all_prime_factors_in(n, lo, hi) and m.divides(n) implies all_prime_factors_in(m, lo, hi)
} by {
    if all_prime_factors_in(n, lo, hi) and m.divides(n) {
        forall(p: Nat) {
            if p.is_prime and p.divides(m) {
                divides_trans(p, m, n)
                p.divides(n)
                all_prime_factors_in_apply(n, lo, hi, p)
                lo < p and p <= hi
            }
            p.is_prime and p.divides(m) implies lo < p and p <= hi
        }
        all_prime_factors_in_intro(m, lo, hi)
        all_prime_factors_in(m, lo, hi)
    }
}

/// An empty interval forces the number to have no prime factors at all.
theorem no_prime_factor_of_empty_interval(n: Nat, lo: Nat, hi: Nat, p: Nat) {
    all_prime_factors_in(n, lo, hi) and hi <= lo and p.is_prime implies not p.divides(n)
} by {
    if all_prime_factors_in(n, lo, hi) and hi <= lo and p.is_prime {
        if p.divides(n) {
            all_prime_factors_in_apply(n, lo, hi, p)
            lo < p
            p <= hi
            lte_trans(p, hi, lo)
            p <= lo
            false
        }
        not p.divides(n)
    }
}
