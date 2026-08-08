from nat import Nat
from algebra.comm_monoid import CommMonoid
from algebra.comm_monoid_rearrange import mul_swap_inner

numerals Nat

/// The product of `f(0), ..., f(n - 1)`.
///
/// The multiplicative counterpart of `range_sum` in `src/nat_range_sum.ac`, indexed by an
/// initial range of the naturals so that the recurrence is available directly and induction on
/// the upper limit needs no set machinery.
define range_prod[M: CommMonoid](f: Nat -> M, n: Nat) -> M {
    match n {
        Nat.zero {
            M.1
        }
        Nat.suc(k) {
            range_prod(f, k) * f(k)
        }
    }
}

/// The empty range has product one.
theorem range_prod_zero[M: CommMonoid](f: Nat -> M) {
    range_prod(f, Nat.0) = M.1
}

/// The standard recurrence: one more factor is multiplied on at the top.
theorem range_prod_suc[M: CommMonoid](f: Nat -> M, k: Nat) {
    range_prod(f, k.suc) = range_prod(f, k) * f(k)
}

/// A range of one factor is that factor.
theorem range_prod_one[M: CommMonoid](f: Nat -> M) {
    range_prod(f, Nat.1) = f(Nat.0)
} by {
    range_prod_suc(f, Nat.0)
    (range_prod(f, Nat.1) = range_prod(f, Nat.0) * f(Nat.0))
    range_prod_zero(f)
    (range_prod(f, Nat.1) = M.1 * f(Nat.0))
    (M.1 * f(Nat.0) = f(Nat.0))
    range_prod(f, Nat.1) = f(Nat.0)
}

/// Factors agreeing below the limit give equal products.
///
/// The product depends on the function only through its values on the range, which is what lets
/// a factor be replaced by any convenient description of it.
theorem range_prod_congr[M: CommMonoid](f: Nat -> M, g: Nat -> M, n: Nat) {
    (forall(i: Nat) { i < n implies f(i) = g(i) }) implies range_prod(f, n) = range_prod(g, n)
} by {
    if forall(i: Nat) { i < n implies f(i) = g(i) } {
        define p(x: Nat) -> Bool {
            x <= n implies range_prod(f, x) = range_prod(g, x)
        }
        range_prod_zero(f)
        range_prod_zero(g)
        p(Nat.0)
        forall(k: Nat) {
            if p(k) {
                if k.suc <= n {
                    k < k.suc
                    k < n
                    f(k) = g(k)
                    k <= n
                    (range_prod(f, k) = range_prod(g, k))
                    range_prod_suc(f, k)
                    range_prod_suc(g, k)
                    range_prod(f, k.suc) = range_prod(g, k.suc)
                }
                p(k.suc)
            }
            (p(k) implies p(k.suc))
        }
        p(Nat.0) and forall(k: Nat) {
            p(k) implies p(k.suc)
        }
        Nat.induction(p)
        p(n)
        n <= n
        range_prod(f, n) = range_prod(g, n)
    }
}

/// The factor read at index `i` as the value at `a + i`.
///
/// The reindexing map. `shift_fn` in `src/nat_range_sum.ac` is the same map but carries an
/// additive constraint it does not use, so a multiplicative range needs its own.
define prod_shift_fn[M: CommMonoid](f: Nat -> M, a: Nat, i: Nat) -> M {
    f(a + i)
}

/// A range product splits at an arbitrary midpoint.
///
/// The upper part is a range product of the shifted factor, so the identity stays within range
/// products rather than needing a notion of a product over an interval.
theorem range_prod_split[M: CommMonoid](f: Nat -> M, m: Nat, n: Nat) {
    range_prod(f, m + n) = range_prod(f, m) * range_prod(prod_shift_fn(f, m), n)
} by {
    define p(x: Nat) -> Bool {
        range_prod(f, m + x) = range_prod(f, m) * range_prod(prod_shift_fn(f, m), x)
    }
    (m + Nat.0 = m)
    range_prod_zero(prod_shift_fn(f, m))
    (range_prod(prod_shift_fn(f, m), Nat.0) = M.1)
    (range_prod(f, m) * M.1 = range_prod(f, m))
    p(Nat.0)
    forall(k: Nat) {
        if p(k) {
            (range_prod(f, m + k) = range_prod(f, m) * range_prod(prod_shift_fn(f, m), k))
            (m + k.suc = (m + k).suc)
            range_prod_suc(f, m + k)
            (range_prod(f, (m + k).suc) = range_prod(f, m + k) * f(m + k))
            range_prod_suc(prod_shift_fn(f, m), k)
            (range_prod(prod_shift_fn(f, m), k.suc)
                = range_prod(prod_shift_fn(f, m), k) * prod_shift_fn(f, m)(k))
            (prod_shift_fn(f, m, k) = f(m + k))
            ((range_prod(f, m) * range_prod(prod_shift_fn(f, m), k)) * f(m + k)
                = range_prod(f, m) * (range_prod(prod_shift_fn(f, m), k) * f(m + k)))
            (range_prod(f, m + k.suc)
                = range_prod(f, m) * range_prod(prod_shift_fn(f, m), k.suc))
            p(k.suc)
        }
        (p(k) implies p(k.suc))
    }
    p(Nat.0) and forall(k: Nat) {
        p(k) implies p(k.suc)
    }
    Nat.induction(p)
    p(n)
}

/// The pointwise product of two factors.
define mul_fn[M: CommMonoid](f: Nat -> M, g: Nat -> M, i: Nat) -> M {
    f(i) * g(i)
}

/// A range product of a pointwise product splits.
theorem range_prod_mul[M: CommMonoid](f: Nat -> M, g: Nat -> M, n: Nat) {
    range_prod(mul_fn(f, g), n) = range_prod(f, n) * range_prod(g, n)
} by {
    define p(x: Nat) -> Bool {
        range_prod(mul_fn(f, g), x) = range_prod(f, x) * range_prod(g, x)
    }
    range_prod_zero(mul_fn(f, g))
    range_prod_zero(f)
    range_prod_zero(g)
    (M.1 * M.1 = M.1)
    p(Nat.0)
    forall(k: Nat) {
        if p(k) {
            (range_prod(mul_fn(f, g), k) = range_prod(f, k) * range_prod(g, k))
            range_prod_suc(mul_fn(f, g), k)
            (range_prod(mul_fn(f, g), k.suc) = range_prod(mul_fn(f, g), k) * mul_fn(f, g)(k))
            (mul_fn(f, g, k) = f(k) * g(k))
            mul_swap_inner[M](range_prod(f, k), range_prod(g, k), f(k), g(k))
            ((range_prod(f, k) * range_prod(g, k)) * (f(k) * g(k))
                = (range_prod(f, k) * f(k)) * (range_prod(g, k) * g(k)))

            range_prod_suc(f, k)
            range_prod_suc(g, k)
            range_prod(mul_fn(f, g), k.suc) = range_prod(f, k.suc) * range_prod(g, k.suc)
            p(k.suc)
        }
        (p(k) implies p(k.suc))
    }
    p(Nat.0) and forall(k: Nat) {
        p(k) implies p(k.suc)
    }
    Nat.induction(p)
    p(n)
}

/// The factor that is one everywhere.
define one_fn[M: CommMonoid](i: Nat) -> M {
    M.1
}

/// The one factor has product one over any range.
theorem range_prod_one_fn[M: CommMonoid](n: Nat) {
    range_prod(one_fn[M], n) = M.1
} by {
    define p(x: Nat) -> Bool {
        range_prod(one_fn[M], x) = M.1
    }
    range_prod_zero(one_fn[M])
    p(Nat.0)
    forall(k: Nat) {
        if p(k) {
            (range_prod(one_fn[M], k) = M.1)
            range_prod_suc(one_fn[M], k)
            (range_prod(one_fn[M], k.suc) = range_prod(one_fn[M], k) * one_fn[M](k))
            (one_fn[M](k) = M.1)
            (M.1 * M.1 = M.1)
            range_prod(one_fn[M], k.suc) = M.1
            p(k.suc)
        }
        (p(k) implies p(k.suc))
    }
    p(Nat.0) and forall(k: Nat) {
        p(k) implies p(k.suc)
    }
    Nat.induction(p)
    p(n)
}

/// Every factor divides the range product.
///
/// The fact a divisibility argument over an interval runs on: a product over a range is a
/// multiple of each of its factors. Induction on the limit, splitting on whether the index is
/// the new one or an earlier one.
theorem range_prod_divides_term(f: Nat -> Nat, n: Nat, k: Nat) {
    k < n implies f(k).divides(range_prod(f, n))
} by {
    define p(x: Nat) -> Bool {
        k < x implies f(k).divides(range_prod(f, x))
    }
    not (k < Nat.0)
    p(Nat.0)
    forall(j: Nat) {
        if p(j) {
            if k < j.suc {
                range_prod_suc(f, j)
                (range_prod(f, j.suc) = range_prod(f, j) * f(j))
                k <= j
                if k = j {
                    (f(k) = f(j))
                    (f(k) * range_prod(f, j) = range_prod(f, j) * f(k))
                    (f(k) * range_prod(f, j) = range_prod(f, j) * f(j))
                    (f(k) * range_prod(f, j) = range_prod(f, j.suc))
                    exists(c: Nat) {
                        f(k) * c = range_prod(f, j.suc)
                    }
                    (f(k).divides(range_prod(f, j.suc))
                        = exists(c: Nat) { f(k) * c = range_prod(f, j.suc) })
                    f(k).divides(range_prod(f, j.suc))
                }
                if k != j {
                    k < j
                    (k < j implies f(k).divides(range_prod(f, j)))
                    f(k).divides(range_prod(f, j))
                    (f(k).divides(range_prod(f, j))
                        = exists(c: Nat) { f(k) * c = range_prod(f, j) })
                    let (d: Nat) satisfy {
                        f(k) * d = range_prod(f, j)
                    }
                    (f(k) * (d * f(j)) = (f(k) * d) * f(j))
                    ((f(k) * d) * f(j) = range_prod(f, j) * f(j))
                    (f(k) * (d * f(j)) = range_prod(f, j.suc))
                    exists(c: Nat) {
                        f(k) * c = range_prod(f, j.suc)
                    }
                    (f(k).divides(range_prod(f, j.suc))
                        = exists(c: Nat) { f(k) * c = range_prod(f, j.suc) })
                    f(k).divides(range_prod(f, j.suc))
                }
                f(k).divides(range_prod(f, j.suc))
            }
            p(j.suc)
        }
        (p(j) implies p(j.suc))
    }
    p(Nat.0) and forall(j: Nat) {
        p(j) implies p(j.suc)
    }
    Nat.induction(p)
    p(n)
}

/// A range product of positive factors is positive.
theorem range_prod_positive(f: Nat -> Nat, n: Nat) {
    (forall(i: Nat) { i < n implies Nat.0 < f(i) }) implies Nat.0 < range_prod(f, n)
} by {
    if forall(i: Nat) { i < n implies Nat.0 < f(i) } {
        define p(x: Nat) -> Bool {
            x <= n implies Nat.0 < range_prod(f, x)
        }
        range_prod_zero(f)
        (range_prod(f, Nat.0) = Nat.1)
        Nat.0 < Nat.1
        p(Nat.0)
        forall(j: Nat) {
            if p(j) {
                if j.suc <= n {
                    j < j.suc
                    j < n
                    Nat.0 < f(j)
                    j <= n
                    Nat.0 < range_prod(f, j)
                    range_prod_suc(f, j)
                    (range_prod(f, j.suc) = range_prod(f, j) * f(j))
                    range_prod(f, j) != Nat.0
                    f(j) != Nat.0
                    range_prod(f, j) * f(j) != Nat.0
                    Nat.0 < range_prod(f, j.suc)
                }
                p(j.suc)
            }
            (p(j) implies p(j.suc))
        }
        p(Nat.0) and forall(j: Nat) {
            p(j) implies p(j.suc)
        }
        Nat.induction(p)
        p(n)
        n <= n
        Nat.0 < range_prod(f, n)
    }
}
