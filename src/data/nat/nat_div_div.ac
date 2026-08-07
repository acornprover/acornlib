from nat import Nat, div_mod_decomp, mod_lt, div_of_decomp, mod_of_decomp, lt_add_left,
    lt_and_lte, lte_mul_both, pow_zero, pow_one, pow_add

numerals Nat

/// Dividing twice is dividing by the product.
///
/// The iterated division law, which `src/nat/` does not state. Writing each division through its
/// decomposition turns the two-step quotient into a single one, and the combined remainder is
/// below the product because the inner one is below its own modulus by at least one step.
theorem nat_div_div(n: Nat, a: Nat, b: Nat) {
    Nat.0 < a and Nat.0 < b implies n.div(a * b) = (n.div(a)).div(b)
} by {
    if Nat.0 < a and Nat.0 < b {
        a != Nat.0
        b != Nat.0
        div_mod_decomp(n, a)
        (n.div(a) * a + n.mod(a) = n)
        mod_lt(n, a)
        n.mod(a) < a
        div_mod_decomp(n.div(a), b)
        ((n.div(a)).div(b) * b + (n.div(a)).mod(b) = n.div(a))
        mod_lt(n.div(a), b)
        (n.div(a)).mod(b) < b
        (((n.div(a)).div(b) * b + (n.div(a)).mod(b)) * a + n.mod(a) = n)
        ((((n.div(a)).div(b) * b + (n.div(a)).mod(b)) * a
            = ((n.div(a)).div(b) * b) * a + (n.div(a)).mod(b) * a))
        (((n.div(a)).div(b) * b) * a = (n.div(a)).div(b) * (b * a))
        (b * a = a * b)
        (((n.div(a)).div(b) * b) * a = (n.div(a)).div(b) * (a * b))
        (((n.div(a)).div(b) * (a * b) + (n.div(a)).mod(b) * a) + n.mod(a)
            = (n.div(a)).div(b) * (a * b) + ((n.div(a)).mod(b) * a + n.mod(a)))
        ((n.div(a)).div(b) * (a * b) + ((n.div(a)).mod(b) * a + n.mod(a)) = n)
        ((n.div(a)).mod(b) + Nat.1 <= b)
        lte_mul_both(a, (n.div(a)).mod(b) + Nat.1, b)
        (a * ((n.div(a)).mod(b) + Nat.1) <= a * b)
        (a * ((n.div(a)).mod(b) + Nat.1) = (n.div(a)).mod(b) * a + a)
        lt_add_left((n.div(a)).mod(b) * a, n.mod(a), a)
        ((n.div(a)).mod(b) * a + n.mod(a) < (n.div(a)).mod(b) * a + a)
        lt_and_lte((n.div(a)).mod(b) * a + n.mod(a),
            (n.div(a)).mod(b) * a + a, a * b)
        ((n.div(a)).mod(b) * a + n.mod(a) < a * b)
        div_of_decomp((n.div(a)).div(b), (n.div(a)).mod(b) * a + n.mod(a), a * b)
        (((n.div(a)).div(b) * (a * b) + ((n.div(a)).mod(b) * a + n.mod(a))).div(a * b)
            = (n.div(a)).div(b))
        n.div(a * b) = (n.div(a)).div(b)
    }
}

/// The remainder of the product division, in terms of the two steps.
///
/// The companion of the quotient law: the combined remainder is the inner remainder plus the
/// outer one scaled by the inner modulus.
theorem nat_mod_mul(n: Nat, a: Nat, b: Nat) {
    Nat.0 < a and Nat.0 < b implies
        n.mod(a * b) = (n.div(a)).mod(b) * a + n.mod(a)
} by {
    if Nat.0 < a and Nat.0 < b {
        a != Nat.0
        b != Nat.0
        div_mod_decomp(n, a)
        (n.div(a) * a + n.mod(a) = n)
        mod_lt(n, a)
        n.mod(a) < a
        div_mod_decomp(n.div(a), b)
        ((n.div(a)).div(b) * b + (n.div(a)).mod(b) = n.div(a))
        mod_lt(n.div(a), b)
        (n.div(a)).mod(b) < b
        (((n.div(a)).div(b) * b + (n.div(a)).mod(b)) * a + n.mod(a) = n)
        ((((n.div(a)).div(b) * b + (n.div(a)).mod(b)) * a
            = ((n.div(a)).div(b) * b) * a + (n.div(a)).mod(b) * a))
        (((n.div(a)).div(b) * b) * a = (n.div(a)).div(b) * (b * a))
        (b * a = a * b)
        (((n.div(a)).div(b) * b) * a = (n.div(a)).div(b) * (a * b))
        (((n.div(a)).div(b) * (a * b) + (n.div(a)).mod(b) * a) + n.mod(a)
            = (n.div(a)).div(b) * (a * b) + ((n.div(a)).mod(b) * a + n.mod(a)))
        ((n.div(a)).div(b) * (a * b) + ((n.div(a)).mod(b) * a + n.mod(a)) = n)
        ((n.div(a)).mod(b) + Nat.1 <= b)
        lte_mul_both(a, (n.div(a)).mod(b) + Nat.1, b)
        (a * ((n.div(a)).mod(b) + Nat.1) <= a * b)
        (a * ((n.div(a)).mod(b) + Nat.1) = (n.div(a)).mod(b) * a + a)
        lt_add_left((n.div(a)).mod(b) * a, n.mod(a), a)
        ((n.div(a)).mod(b) * a + n.mod(a) < (n.div(a)).mod(b) * a + a)
        lt_and_lte((n.div(a)).mod(b) * a + n.mod(a),
            (n.div(a)).mod(b) * a + a, a * b)
        ((n.div(a)).mod(b) * a + n.mod(a) < a * b)
        mod_of_decomp((n.div(a)).div(b), (n.div(a)).mod(b) * a + n.mod(a), a * b)
        (((n.div(a)).div(b) * (a * b) + ((n.div(a)).mod(b) * a + n.mod(a))).mod(a * b)
            = (n.div(a)).mod(b) * a + n.mod(a))
        n.mod(a * b) = (n.div(a)).mod(b) * a + n.mod(a)
    }
}

/// The order of the two divisions does not matter.
theorem nat_div_div_comm(n: Nat, a: Nat, b: Nat) {
    Nat.0 < a and Nat.0 < b implies (n.div(a)).div(b) = (n.div(b)).div(a)
} by {
    if Nat.0 < a and Nat.0 < b {
        nat_div_div(n, a, b)
        (n.div(a * b) = (n.div(a)).div(b))
        nat_div_div(n, b, a)
        (n.div(b * a) = (n.div(b)).div(a))
        (a * b = b * a)
        (n.div(a)).div(b) = (n.div(b)).div(a)
    }
}

/// A power of a positive natural is positive.
theorem nat_pow_pos(b: Nat, k: Nat) {
    Nat.0 < b implies Nat.0 < b.pow(k)
} by {
    if Nat.0 < b {
        define p(x: Nat) -> Bool {
            Nat.0 < b.pow(x)
        }
        pow_zero[Nat](b)
        (b.pow(Nat.0) = Nat.1)
        Nat.0 < Nat.1
        p(Nat.0)
        forall(x: Nat) {
            if p(x) {
                Nat.0 < b.pow(x)
                pow_one[Nat](b)
                (b.pow(Nat.1) = b)
                pow_add[Nat](b, x, Nat.1)
                (b.pow(x + Nat.1) = b.pow(x) * b.pow(Nat.1))
                (x + Nat.1 = x.suc)
                (b.pow(x.suc) = b.pow(x) * b)
                b.pow(x) != Nat.0
                b != Nat.0
                b.pow(x) * b != Nat.0
                Nat.0 < b.pow(x.suc)
                p(x.suc)
            }
            (p(x) implies p(x.suc))
        }
        p(Nat.0) and forall(x: Nat) {
            p(x) implies p(x.suc)
        }
        Nat.induction(p)
        p(k)
    }
}

/// Dividing by one more power is dividing once more by the base.
theorem nat_div_pow_suc(b: Nat, n: Nat, k: Nat) {
    Nat.0 < b implies n.div(b.pow(k.suc)) = (n.div(b.pow(k))).div(b)
} by {
    if Nat.0 < b {
        nat_pow_pos(b, k)
        Nat.0 < b.pow(k)
        pow_one[Nat](b)
        (b.pow(Nat.1) = b)
        pow_add[Nat](b, k, Nat.1)
        (b.pow(k + Nat.1) = b.pow(k) * b.pow(Nat.1))
        (k + Nat.1 = k.suc)
        (b.pow(k.suc) = b.pow(k) * b)
        nat_div_div(n, b.pow(k), b)
        (n.div(b.pow(k) * b) = (n.div(b.pow(k))).div(b))
        n.div(b.pow(k.suc)) = (n.div(b.pow(k))).div(b)
    }
}

/// The `k`th digit of a natural in base `b`.
///
/// The companion of `real_digit` in `src/real_base_digits.ac`, defined the way a base expansion
/// of a natural is usually read: shift past the lower places and take the remainder.
define nat_digit(b: Nat, n: Nat, k: Nat) -> Nat {
    (n.div(b.pow(k))).mod(b)
}

/// Every digit lies below the base.
theorem nat_digit_lt_base(b: Nat, n: Nat, k: Nat) {
    Nat.0 < b implies nat_digit(b, n, k) < b
} by {
    if Nat.0 < b {
        b != Nat.0
        mod_lt(n.div(b.pow(k)), b)
        ((n.div(b.pow(k))).mod(b) < b)
        nat_digit(b, n, k) < b
    }
}

/// A digit is what one more division leaves behind.
///
/// The recursion the agreement with the real digits runs on: the place value at `k` is recovered
/// from the quotient at `k` and the quotient at `k + 1`.
theorem nat_digit_eq_div_sub(b: Nat, n: Nat, k: Nat) {
    Nat.0 < b implies
        (n.div(b.pow(k))).div(b) * b + nat_digit(b, n, k) = n.div(b.pow(k))
} by {
    if Nat.0 < b {
        div_mod_decomp(n.div(b.pow(k)), b)
        ((n.div(b.pow(k))).div(b) * b + (n.div(b.pow(k))).mod(b) = n.div(b.pow(k)))
        (nat_digit(b, n, k) = (n.div(b.pow(k))).mod(b))
        ((n.div(b.pow(k))).div(b) * b + nat_digit(b, n, k) = n.div(b.pow(k)))
    }
}
