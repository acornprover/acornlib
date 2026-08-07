from nat import Nat, lte_trans, lte_and_lt, lt_and_lte, lte_suc_suc, sum_lte,
    lt_or_lte, lt_imp_lte_suc, lte_antisymm

numerals Nat

/// The number of naturals below `n` satisfying `p`.
///
/// The counting function of a set of naturals, indexed by an initial range so that the
/// recurrence is available directly. Densities and almost-all quantification are both stated
/// in terms of it.
define count_upto(p: Nat -> Bool, n: Nat) -> Nat {
    match n {
        Nat.zero {
            Nat.0
        }
        Nat.suc(k) {
            if p(k) {
                count_upto(p, k) + Nat.1
            } else {
                count_upto(p, k)
            }
        }
    }
}

/// Nothing lies below zero.
theorem count_upto_zero(p: Nat -> Bool) {
    count_upto(p, Nat.0) = Nat.0
}

/// Extending the range by a satisfying value increments the count.
theorem count_upto_suc_true(p: Nat -> Bool, k: Nat) {
    p(k) implies count_upto(p, k.suc) = count_upto(p, k) + Nat.1
} by {
    if p(k) {
        count_upto(p, k.suc) = count_upto(p, k) + Nat.1
    }
}

/// Extending the range by a failing value leaves the count alone.
theorem count_upto_suc_false(p: Nat -> Bool, k: Nat) {
    not p(k) implies count_upto(p, k.suc) = count_upto(p, k)
} by {
    if not p(k) {
        count_upto(p, k.suc) = count_upto(p, k)
    }
}

/// Extending the range by one changes the count by at most one.
theorem count_upto_suc_bounds(p: Nat -> Bool, k: Nat) {
    count_upto(p, k) <= count_upto(p, k.suc)
        and count_upto(p, k.suc) <= count_upto(p, k) + Nat.1
} by {
    if p(k) {
        count_upto_suc_true(p, k)
        count_upto(p, k.suc) = count_upto(p, k) + Nat.1
        count_upto(p, k) <= count_upto(p, k.suc)
        count_upto(p, k.suc) <= count_upto(p, k) + Nat.1
    }
    if not p(k) {
        count_upto_suc_false(p, k)
        count_upto(p, k.suc) = count_upto(p, k)
        count_upto(p, k) <= count_upto(p, k.suc)
        count_upto(p, k) <= count_upto(p, k) + Nat.1
        count_upto(p, k.suc) <= count_upto(p, k) + Nat.1
    }
    (count_upto(p, k) <= count_upto(p, k.suc)
        and count_upto(p, k.suc) <= count_upto(p, k) + Nat.1)
}

/// The count never exceeds the size of the range.
///
/// Each step of the range adds at most one to the count.
theorem count_upto_le(p: Nat -> Bool, n: Nat) {
    count_upto(p, n) <= n
} by {
    define q(x: Nat) -> Bool {
        count_upto(p, x) <= x
    }
    count_upto_zero(p)
    count_upto(p, Nat.0) <= Nat.0
    q(Nat.0)
    forall(k: Nat) {
        if q(k) {
            count_upto(p, k) <= k
            count_upto_suc_bounds(p, k)
            count_upto(p, k.suc) <= count_upto(p, k) + Nat.1
            count_upto(p, k) + Nat.1 <= k + Nat.1
            lte_trans(count_upto(p, k.suc), count_upto(p, k) + Nat.1, k + Nat.1)
            count_upto(p, k.suc) <= k + Nat.1
            k + Nat.1 = k.suc
            count_upto(p, k.suc) <= k.suc
            q(k.suc)
        }
        (q(k) implies q(k.suc))
    }
    q(Nat.0) and forall(k: Nat) {
        q(k) implies q(k.suc)
    }
    Nat.induction(q)
    q(n)
}

/// The count is nondecreasing in the range.
theorem count_upto_mono(p: Nat -> Bool, m: Nat, n: Nat) {
    m <= n implies count_upto(p, m) <= count_upto(p, n)
} by {
    define q(x: Nat) -> Bool {
        m <= x implies count_upto(p, m) <= count_upto(p, x)
    }
    if m <= Nat.0 {
        m = Nat.0
        count_upto(p, m) <= count_upto(p, Nat.0)
    }
    q(Nat.0)
    forall(k: Nat) {
        if q(k) {
            if m <= k.suc {
                count_upto_suc_bounds(p, k)
                count_upto(p, k) <= count_upto(p, k.suc)
                if m <= k {
                    count_upto(p, m) <= count_upto(p, k)
                    lte_trans(count_upto(p, m), count_upto(p, k), count_upto(p, k.suc))
                    count_upto(p, m) <= count_upto(p, k.suc)
                }
                if not (m <= k) {
                    lt_or_lte(k, m)
                    k < m
                    lt_imp_lte_suc(k, m)
                    k.suc <= m
                    lte_antisymm(m, k.suc)
                    m = k.suc
                    count_upto(p, m) <= count_upto(p, k.suc)
                }
                count_upto(p, m) <= count_upto(p, k.suc)
            }
            q(k.suc)
        }
        (q(k) implies q(k.suc))
    }
    q(Nat.0) and forall(k: Nat) {
        q(k) implies q(k.suc)
    }
    Nat.induction(q)
    q(n)
}

/// A predicate implied by another counts no more often.
///
/// Only the values below the range matter, so the hypothesis is stated there rather than
/// everywhere.
theorem count_upto_mono_pred(p: Nat -> Bool, r: Nat -> Bool, n: Nat) {
    (forall(i: Nat) { i < n implies (p(i) implies r(i)) })
        implies count_upto(p, n) <= count_upto(r, n)
} by {
    define q(x: Nat) -> Bool {
        (forall(i: Nat) { i < x implies (p(i) implies r(i)) })
            implies count_upto(p, x) <= count_upto(r, x)
    }
    count_upto_zero(p)
    count_upto_zero(r)
    count_upto(p, Nat.0) <= count_upto(r, Nat.0)
    q(Nat.0)
    forall(k: Nat) {
        if q(k) {
            if forall(i: Nat) { i < k.suc implies (p(i) implies r(i)) } {
                forall(i: Nat) {
                    if i < k {
                        i < k.suc
                        (p(i) implies r(i))
                    }
                    (i < k implies (p(i) implies r(i)))
                }
                count_upto(p, k) <= count_upto(r, k)
                k < k.suc
                (p(k) implies r(k))
                if p(k) {
                    r(k)
                    count_upto_suc_true(p, k)
                    count_upto(p, k.suc) = count_upto(p, k) + Nat.1
                    count_upto_suc_true(r, k)
                    count_upto(r, k.suc) = count_upto(r, k) + Nat.1
                    count_upto(p, k) + Nat.1 <= count_upto(r, k) + Nat.1
                    count_upto(p, k.suc) <= count_upto(r, k.suc)
                }
                if not p(k) {
                    count_upto_suc_false(p, k)
                    count_upto(p, k.suc) = count_upto(p, k)
                    count_upto_suc_bounds(r, k)
                    count_upto(r, k) <= count_upto(r, k.suc)
                    lte_trans(count_upto(p, k), count_upto(r, k), count_upto(r, k.suc))
                    count_upto(p, k.suc) <= count_upto(r, k.suc)
                }
                count_upto(p, k.suc) <= count_upto(r, k.suc)
            }
            q(k.suc)
        }
        (q(k) implies q(k.suc))
    }
    q(Nat.0) and forall(k: Nat) {
        q(k) implies q(k.suc)
    }
    Nat.induction(q)
    q(n)
}

/// The pointwise negation of a predicate on the naturals.
define complement_pred(p: Nat -> Bool, i: Nat) -> Bool {
    not p(i)
}

/// A predicate and its complement account for the whole range.
///
/// Every value below the range satisfies exactly one of the two, so the two counts add to the
/// size of the range. This is what turns a density statement into one about the complement.
theorem count_upto_complement(p: Nat -> Bool, n: Nat) {
    count_upto(p, n) + count_upto(complement_pred(p), n) = n
} by {
    define q(x: Nat) -> Bool {
        count_upto(p, x) + count_upto(complement_pred(p), x) = x
    }
    count_upto_zero(p)
    count_upto_zero(complement_pred(p))
    Nat.0 + Nat.0 = Nat.0
    count_upto(p, Nat.0) + count_upto(complement_pred(p), Nat.0) = Nat.0
    q(Nat.0)
    forall(k: Nat) {
        if q(k) {
            count_upto(p, k) + count_upto(complement_pred(p), k) = k
            if p(k) {
                count_upto_suc_true(p, k)
                count_upto(p, k.suc) = count_upto(p, k) + Nat.1
                complement_pred(p, k) = not p(k)
                not complement_pred(p)(k)
                count_upto_suc_false(complement_pred(p), k)
                count_upto(complement_pred(p), k.suc) = count_upto(complement_pred(p), k)
                (count_upto(p, k.suc) + count_upto(complement_pred(p), k.suc)
                    = (count_upto(p, k) + Nat.1) + count_upto(complement_pred(p), k))
                ((count_upto(p, k) + Nat.1) + count_upto(complement_pred(p), k)
                    = (count_upto(p, k) + count_upto(complement_pred(p), k)) + Nat.1)
                count_upto(p, k.suc) + count_upto(complement_pred(p), k.suc) = k + Nat.1
            }
            if not p(k) {
                count_upto_suc_false(p, k)
                count_upto(p, k.suc) = count_upto(p, k)
                complement_pred(p, k) = not p(k)
                complement_pred(p)(k)
                count_upto_suc_true(complement_pred(p), k)
                (count_upto(complement_pred(p), k.suc)
                    = count_upto(complement_pred(p), k) + Nat.1)
                (count_upto(p, k.suc) + count_upto(complement_pred(p), k.suc)
                    = count_upto(p, k) + (count_upto(complement_pred(p), k) + Nat.1))
                (count_upto(p, k) + (count_upto(complement_pred(p), k) + Nat.1)
                    = (count_upto(p, k) + count_upto(complement_pred(p), k)) + Nat.1)
                count_upto(p, k.suc) + count_upto(complement_pred(p), k.suc) = k + Nat.1
            }
            k + Nat.1 = k.suc
            count_upto(p, k.suc) + count_upto(complement_pred(p), k.suc) = k.suc
            q(k.suc)
        }
        (q(k) implies q(k.suc))
    }
    q(Nat.0) and forall(k: Nat) {
        q(k) implies q(k.suc)
    }
    Nat.induction(q)
    q(n)
}

/// The predicate that holds everywhere.
define all_pred(i: Nat) -> Bool {
    true
}

/// Everything below the range satisfies the everywhere-true predicate.
theorem count_upto_all(n: Nat) {
    count_upto(all_pred, n) = n
} by {
    define q(x: Nat) -> Bool {
        count_upto(all_pred, x) = x
    }
    count_upto_zero(all_pred)
    q(Nat.0)
    forall(k: Nat) {
        if q(k) {
            count_upto(all_pred, k) = k
            all_pred(k)
            count_upto_suc_true(all_pred, k)
            count_upto(all_pred, k.suc) = count_upto(all_pred, k) + Nat.1
            count_upto(all_pred, k.suc) = k + Nat.1
            k + Nat.1 = k.suc
            count_upto(all_pred, k.suc) = k.suc
            q(k.suc)
        }
        (q(k) implies q(k.suc))
    }
    q(Nat.0) and forall(k: Nat) {
        q(k) implies q(k.suc)
    }
    Nat.induction(q)
    q(n)
}

/// The predicate that holds nowhere.
define none_pred(i: Nat) -> Bool {
    false
}

/// Nothing satisfies the nowhere-true predicate.
theorem count_upto_none(n: Nat) {
    count_upto(none_pred, n) = Nat.0
} by {
    define q(x: Nat) -> Bool {
        count_upto(none_pred, x) = Nat.0
    }
    count_upto_zero(none_pred)
    q(Nat.0)
    forall(k: Nat) {
        if q(k) {
            count_upto(none_pred, k) = Nat.0
            not none_pred(k)
            count_upto_suc_false(none_pred, k)
            count_upto(none_pred, k.suc) = count_upto(none_pred, k)
            count_upto(none_pred, k.suc) = Nat.0
            q(k.suc)
        }
        (q(k) implies q(k.suc))
    }
    q(Nat.0) and forall(k: Nat) {
        q(k) implies q(k.suc)
    }
    Nat.induction(q)
    q(n)
}
