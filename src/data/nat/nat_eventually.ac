/// Generic tail-eventual predicates over natural numbers.
///
/// This module intentionally does not define asymptotic notation.  It supplies the
/// reusable `exists n0, forall n >= n0` support layer expected by downstream
/// big-O/little-o APIs.

from nat import Nat, lte_ref, lte_trans
from order import max_imp_gte
from data.basic.set import Set, nat_from, nat_from_contains_eq, subset_contains,
    intersection_contains_intro, intersection_contains_eq
from data.basic.relation_transport import predicate_subset, predicate_subset_step,
    predicate_subset_refl

/// True if predicate `p` holds at every index at least `n0`.
define eventually_after_nat(p: Nat -> Bool, n0: Nat) -> Bool {
    forall(n: Nat) {
        n0 <= n implies p(n)
    }
}

/// True if predicate `p` holds for all sufficiently large natural numbers.
define eventually_predicate_nat(p: Nat -> Bool) -> Bool {
    exists(n0: Nat) {
        eventually_after_nat(p, n0)
    }
}

/// True if a set of natural numbers contains a final segment.
define eventually_in_nat_set(s: Set[Nat]) -> Bool {
    eventually_predicate_nat(s.contains)
}

/// Pointwise conjunction of two Nat predicates.
define predicate_and_nat(p: Nat -> Bool, q: Nat -> Bool, n: Nat) -> Bool {
    p(n) and q(n)
}

/// The always-true Nat predicate.
define predicate_true_nat(n: Nat) -> Bool {
    true
}

/// An eventual-tail hypothesis applies at any sufficiently late index.
theorem eventually_after_nat_at(p: Nat -> Bool, n0: Nat, n: Nat) {
    eventually_after_nat(p, n0) and n0 <= n implies p(n)
} by {
    if eventually_after_nat(p, n0) and n0 <= n {
        eventually_after_nat(p, n0) = forall(k: Nat) {
            n0 <= k implies p(k)
        }
        p(n)
    }
}

/// A concrete tail witness proves eventuality.
theorem eventually_predicate_nat_intro(p: Nat -> Bool, n0: Nat) {
    eventually_after_nat(p, n0) implies eventually_predicate_nat(p)
} by {
    if eventually_after_nat(p, n0) {
        exists(m: Nat) {
            eventually_after_nat(p, m)
        }
    }
}

/// Eventuality yields a concrete tail witness.
theorem eventually_predicate_nat_has_witness(p: Nat -> Bool) {
    eventually_predicate_nat(p) implies exists(n0: Nat) {
        eventually_after_nat(p, n0)
    }
} by {
    if eventually_predicate_nat(p) {
        exists(n0: Nat) {
            eventually_after_nat(p, n0)
        }
    }
}

/// A predicate true at every index is eventually true.
theorem eventually_predicate_nat_of_forall(p: Nat -> Bool) {
    (forall(n: Nat) { p(n) }) implies eventually_predicate_nat(p)
} by {
    if forall(n: Nat) { p(n) } {
        forall(n: Nat) {
            if Nat.0 <= n {
                p(n)
            }
        }
        eventually_after_nat(p, Nat.0)
        eventually_predicate_nat_intro(p, Nat.0)
        eventually_predicate_nat(p)
    }
}

/// The always-true predicate is eventually true.
theorem eventually_predicate_nat_true {
    eventually_predicate_nat(predicate_true_nat)
} by {
    forall(n: Nat) {
        predicate_true_nat(n)
    }
    eventually_predicate_nat_of_forall(predicate_true_nat)
}

/// Moving the start of a valid tail later preserves validity of the tail.
theorem eventually_after_nat_strengthen(p: Nat -> Bool, n0: Nat, n1: Nat) {
    eventually_after_nat(p, n0) and n0 <= n1 implies eventually_after_nat(p, n1)
} by {
    if eventually_after_nat(p, n0) and n0 <= n1 {
        forall(n: Nat) {
            if n1 <= n {
                lte_trans(n0, n1, n)
                n0 <= n
                eventually_after_nat_at(p, n0, n)
                p(n)
            }
        }
    }
}

/// Eventuality is preserved by pointwise predicate inclusion.
theorem eventually_predicate_nat_monotone(p: Nat -> Bool, q: Nat -> Bool) {
    predicate_subset(p, q) and eventually_predicate_nat(p) implies eventually_predicate_nat(q)
} by {
    if predicate_subset(p, q) and eventually_predicate_nat(p) {
        eventually_predicate_nat_has_witness(p)
        let n0: Nat satisfy {
            eventually_after_nat(p, n0)
        }
        forall(n: Nat) {
            if n0 <= n {
                eventually_after_nat_at(p, n0, n)
                p(n)
                predicate_subset_step(p, q, n)
                q(n)
            }
        }
        eventually_after_nat(q, n0)
        eventually_predicate_nat_intro(q, n0)
        eventually_predicate_nat(q)
    }
}

/// Eventuality respects equality of predicates.
theorem eventually_predicate_nat_transport_eq(p: Nat -> Bool, q: Nat -> Bool) {
    p = q and eventually_predicate_nat(p) implies eventually_predicate_nat(q)
} by {
    if p = q and eventually_predicate_nat(p) {
        predicate_subset_refl(p)
        predicate_subset(p, q)
        eventually_predicate_nat_monotone(p, q)
        eventually_predicate_nat(q)
    }
}

/// Two eventually true predicates are eventually true together.
theorem eventually_predicate_nat_conjunction(p: Nat -> Bool, q: Nat -> Bool) {
    eventually_predicate_nat(p) and eventually_predicate_nat(q)
    implies eventually_predicate_nat(predicate_and_nat(p, q))
} by {
    if eventually_predicate_nat(p) and eventually_predicate_nat(q) {
        eventually_predicate_nat_has_witness(p)
        let np: Nat satisfy {
            eventually_after_nat(p, np)
        }
        eventually_predicate_nat_has_witness(q)
        let nq: Nat satisfy {
            eventually_after_nat(q, nq)
        }
        let n0 = np.max(nq)
        max_imp_gte[Nat](np, nq)
        np.max(nq) >= np and np.max(nq) >= nq
        np <= n0
        nq <= n0
        forall(n: Nat) {
            if n0 <= n {
                lte_trans(np, n0, n)
                np <= n
                lte_trans(nq, n0, n)
                nq <= n
                eventually_after_nat_at(p, np, n)
                p(n)
                eventually_after_nat_at(q, nq, n)
                q(n)
                predicate_and_nat(p, q, n)
            }
        }
        eventually_after_nat(predicate_and_nat(p, q), n0)
        eventually_predicate_nat_intro(predicate_and_nat(p, q), n0)
        eventually_predicate_nat(predicate_and_nat(p, q))
    }
}

/// Eventual conjunction gives eventuality of the left component.
theorem eventually_predicate_nat_conjunction_left(p: Nat -> Bool, q: Nat -> Bool) {
    eventually_predicate_nat(predicate_and_nat(p, q)) implies eventually_predicate_nat(p)
} by {
    if eventually_predicate_nat(predicate_and_nat(p, q)) {
        eventually_predicate_nat_has_witness(predicate_and_nat(p, q))
        let n0: Nat satisfy {
            eventually_after_nat(predicate_and_nat(p, q), n0)
        }
        forall(n: Nat) {
            if n0 <= n {
                eventually_after_nat_at(predicate_and_nat(p, q), n0, n)
                predicate_and_nat(p, q, n)
                p(n)
            }
        }
        eventually_after_nat(p, n0)
        eventually_predicate_nat_intro(p, n0)
        eventually_predicate_nat(p)
    }
}

/// Eventual conjunction gives eventuality of the right component.
theorem eventually_predicate_nat_conjunction_right(p: Nat -> Bool, q: Nat -> Bool) {
    eventually_predicate_nat(predicate_and_nat(p, q)) implies eventually_predicate_nat(q)
} by {
    if eventually_predicate_nat(predicate_and_nat(p, q)) {
        eventually_predicate_nat_has_witness(predicate_and_nat(p, q))
        let n0: Nat satisfy {
            eventually_after_nat(predicate_and_nat(p, q), n0)
        }
        forall(n: Nat) {
            if n0 <= n {
                eventually_after_nat_at(predicate_and_nat(p, q), n0, n)
                predicate_and_nat(p, q, n)
                q(n)
            }
        }
        eventually_after_nat(q, n0)
        eventually_predicate_nat_intro(q, n0)
        eventually_predicate_nat(q)
    }
}

/// Eventuality is equivalent to containment of a final segment in the predicate set.
theorem eventually_predicate_nat_iff_nat_from_subset(p: Nat -> Bool) {
    eventually_predicate_nat(p) = exists(n0: Nat) {
        nat_from(n0).subset(Set[Nat].new(p))
    }
} by {
    if eventually_predicate_nat(p) {
        eventually_predicate_nat_has_witness(p)
        let n0: Nat satisfy {
            eventually_after_nat(p, n0)
        }
        forall(n: Nat) {
            if nat_from(n0).contains(n) {
                nat_from_contains_eq(n0, n)
                n0 <= n
                eventually_after_nat_at(p, n0, n)
                p(n)
                Set[Nat].new(p).contains(n)
            }
        }
        nat_from(n0).subset(Set[Nat].new(p))
        exists(m: Nat) {
            nat_from(m).subset(Set[Nat].new(p))
        }
    }
    if exists(n0: Nat) { nat_from(n0).subset(Set[Nat].new(p)) } {
        let n0: Nat satisfy {
            nat_from(n0).subset(Set[Nat].new(p))
        }
        forall(n: Nat) {
            if n0 <= n {
                nat_from_contains_eq(n0, n)
                nat_from(n0).contains(n)
                subset_contains(nat_from(n0), Set[Nat].new(p), n)
                Set[Nat].new(p).contains(n)
                p(n)
            }
        }
        eventually_after_nat(p, n0)
        eventually_predicate_nat_intro(p, n0)
        eventually_predicate_nat(p)
    }
    eventually_predicate_nat(p) = exists(n0: Nat) {
        nat_from(n0).subset(Set[Nat].new(p))
    }
}

/// Set-eventuality is equivalent to containing some final Nat segment.
theorem eventually_in_nat_set_iff_nat_from_subset(s: Set[Nat]) {
    eventually_in_nat_set(s) = exists(n0: Nat) {
        nat_from(n0).subset(s)
    }
} by {
    if eventually_in_nat_set(s) {
        eventually_predicate_nat_iff_nat_from_subset(s.contains)
        exists(n0: Nat) {
            nat_from(n0).subset(Set[Nat].new(s.contains))
        }
        let n0: Nat satisfy {
            nat_from(n0).subset(Set[Nat].new(s.contains))
        }
        forall(n: Nat) {
            if nat_from(n0).contains(n) {
                subset_contains(nat_from(n0), Set[Nat].new(s.contains), n)
                Set[Nat].new(s.contains).contains(n)
                s.contains(n)
            }
        }
        nat_from(n0).subset(s)
        exists(m: Nat) {
            nat_from(m).subset(s)
        }
    }
    if exists(n0: Nat) { nat_from(n0).subset(s) } {
        let n0: Nat satisfy {
            nat_from(n0).subset(s)
        }
        forall(n: Nat) {
            if nat_from(n0).contains(n) {
                subset_contains(nat_from(n0), s, n)
                s.contains(n)
                Set[Nat].new(s.contains).contains(n)
            }
        }
        nat_from(n0).subset(Set[Nat].new(s.contains))
        exists(m: Nat) {
            nat_from(m).subset(Set[Nat].new(s.contains))
        }
        eventually_predicate_nat_iff_nat_from_subset(s.contains)
        eventually_predicate_nat(s.contains)
        eventually_in_nat_set(s)
    }
    eventually_in_nat_set(s) = exists(n0: Nat) {
        nat_from(n0).subset(s)
    }
}

/// Set-eventuality is preserved by set inclusion.
theorem eventually_in_nat_set_of_subset(s: Set[Nat], t: Set[Nat]) {
    s.subset(t) and eventually_in_nat_set(s) implies eventually_in_nat_set(t)
} by {
    if s.subset(t) and eventually_in_nat_set(s) {
        eventually_predicate_nat_has_witness(s.contains)
        let n0: Nat satisfy {
            eventually_after_nat(s.contains, n0)
        }
        forall(n: Nat) {
            if n0 <= n {
                eventually_after_nat_at(s.contains, n0, n)
                s.contains(n)
                subset_contains(s, t, n)
                t.contains(n)
            }
        }
        eventually_after_nat(t.contains, n0)
        eventually_predicate_nat_intro(t.contains, n0)
        eventually_predicate_nat(t.contains)
        eventually_in_nat_set(t)
    }
}

/// If two Nat sets are eventually inhabited, their intersection is eventually inhabited.
theorem eventually_in_nat_set_intersection(s: Set[Nat], t: Set[Nat]) {
    eventually_in_nat_set(s) and eventually_in_nat_set(t) implies eventually_in_nat_set(s.intersection(t))
} by {
    if eventually_in_nat_set(s) and eventually_in_nat_set(t) {
        eventually_predicate_nat_has_witness(s.contains)
        let ns: Nat satisfy {
            eventually_after_nat(s.contains, ns)
        }
        eventually_predicate_nat_has_witness(t.contains)
        let nt: Nat satisfy {
            eventually_after_nat(t.contains, nt)
        }
        let n0 = ns.max(nt)
        max_imp_gte[Nat](ns, nt)
        ns.max(nt) >= ns and ns.max(nt) >= nt
        ns <= n0
        nt <= n0
        forall(n: Nat) {
            if n0 <= n {
                lte_trans(ns, n0, n)
                ns <= n
                lte_trans(nt, n0, n)
                nt <= n
                eventually_after_nat_at(s.contains, ns, n)
                s.contains(n)
                eventually_after_nat_at(t.contains, nt, n)
                t.contains(n)
                intersection_contains_intro(s, t, n)
                s.intersection(t).contains(n)
            }
        }
        eventually_after_nat((s.intersection(t)).contains, n0)
        eventually_predicate_nat_intro((s.intersection(t)).contains, n0)
        eventually_predicate_nat((s.intersection(t)).contains)
        eventually_in_nat_set(s.intersection(t))
    }
}
