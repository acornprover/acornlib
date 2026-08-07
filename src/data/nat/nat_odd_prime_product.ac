from nat import Nat, divides_self, divides_lte, lte_and_lt, lt_and_lte, lte_trans, add_imp_sub,
    lte_add_left, lt_add_left
from number_theory import falling_product, falling_product_eq_binom_mul_factorial,
    prime_divides_mul
from combinatorics import binom, choose_symm_add_form, binom_pos
from falling_product_prime_factors import prime_factor_divides_falling_product
from data.nat.nat_prime_central_binom import prime_divides_factorial_imp_lte
from data.nat.nat_range_prod import range_prod
from data.nat.nat_odd_binom_bound import odd_binom_lte_four_pow
from data.nat.nat_prime_interval_product import interval_prime_fn, primes_in_interval_divide,
    interval_prime_product_divides

numerals Nat

/// A prime above `m + 1` and at most `2m + 1` divides the middle coefficient of that odd row.
///
/// The odd companion of `prime_in_interval_divides_central_binom`. The falling product
/// `(2m + 1) * 2m * ... * (m + 1)` has such a prime among its factors, and that product is the
/// coefficient times `(m + 1)!`. The prime is too large to come from the factorial, so it comes
/// from the coefficient, and symmetry moves it to the middle index.
theorem prime_in_interval_divides_odd_binom(p: Nat, m: Nat) {
    p.is_prime and m.suc < p and p <= m + m.suc
        implies p.divides((m + m.suc).binom(m))
} by {
    if p.is_prime and m.suc < p and p <= m + m.suc {
        (p <= m + m.suc) = exists(c: Nat) { p + c = m + m.suc }
        let (i: Nat) satisfy {
            p + i = m + m.suc
        }
        if m <= i {
            lte_add_left(m.suc, m, i)
            m.suc + m <= m.suc + i
            lt_add_left(i, m.suc, p)
            i + m.suc < i + p
            m.suc + i < p + i
            lte_and_lt(m.suc + m, m.suc + i, p + i)
            m.suc + m < p + i
            (m.suc + m = m + m.suc)
            m + m.suc < m + m.suc
            false
        }
        not (m <= i)
        i < m
        i <= m
        i + p = m + m.suc
        add_imp_sub(p, i, m + m.suc)
        (m + m.suc) - i = p
        divides_self(p)
        p.divides((m + m.suc) - i)
        prime_factor_divides_falling_product(m + m.suc, m, p, i)
        p.divides(falling_product(m + m.suc, m))
        m < m.suc
        m.suc <= m + m.suc
        lt_and_lte(m, m.suc, m + m.suc)
        m < m + m.suc
        falling_product_eq_binom_mul_factorial(m + m.suc, m)
        (falling_product(m + m.suc, m)
            = (m + m.suc).binom(m.suc) * m.suc.factorial)
        p.divides((m + m.suc).binom(m.suc) * m.suc.factorial)
        prime_divides_mul(p, (m + m.suc).binom(m.suc), m.suc.factorial)
        (p.divides((m + m.suc).binom(m.suc)) or p.divides(m.suc.factorial))
        if p.divides(m.suc.factorial) {
            prime_divides_factorial_imp_lte(p, m.suc)
            p <= m.suc
            false
        }
        not p.divides(m.suc.factorial)
        p.divides((m + m.suc).binom(m.suc))
        choose_symm_add_form(m, m.suc)
        ((m + m.suc).binom(m) = (m + m.suc).binom(m.suc))
        p.divides((m + m.suc).binom(m))
    }
}

/// The product of the primes above `m + 1` and at most `2m + 1` divides that coefficient.
///
/// The odd companion of `central_binom_prime_interval_product_divides`. The interval runs from
/// `m + 2` for `m` places, which is exactly the primes strictly between `m + 1` and `2m + 2`.
theorem odd_binom_prime_interval_product_divides(m: Nat) {
    range_prod(interval_prime_fn(m.suc.suc), m).divides((m + m.suc).binom(m))
} by {
    forall(i: Nat) {
        if i < m and (m.suc.suc + i).is_prime {
            m.suc < m.suc.suc
            m.suc.suc <= m.suc.suc + i
            lt_and_lte(m.suc, m.suc.suc, m.suc.suc + i)
            m.suc < m.suc.suc + i
            i.suc <= m
            (i + Nat.1 <= m)
            lte_add_left(Nat.1, i + Nat.1, m)
            (Nat.1 + (i + Nat.1) <= Nat.1 + m)
            (Nat.1 + (i + Nat.1) = i + Nat.2)
            (Nat.1 + m = m + Nat.1)
            (i + Nat.2 <= m + Nat.1)
            lte_add_left(m, i + Nat.2, m + Nat.1)
            (m + (i + Nat.2) <= m + (m + Nat.1))
            (m.suc.suc = m + Nat.2)
            ((m + Nat.2) + i = m + (Nat.2 + i))
            (Nat.2 + i = i + Nat.2)
            (m.suc.suc + i = m + (i + Nat.2))
            (m + m.suc = m + (m + Nat.1))
            (m.suc.suc + i <= m + m.suc)
            prime_in_interval_divides_odd_binom(m.suc.suc + i, m)
            (m.suc.suc + i).divides((m + m.suc).binom(m))
        }
        (i < m and (m.suc.suc + i).is_prime
            implies (m.suc.suc + i).divides((m + m.suc).binom(m)))
    }
    (primes_in_interval_divide(m.suc.suc, m, (m + m.suc).binom(m))
        = forall(i: Nat) {
        i < m and (m.suc.suc + i).is_prime
            implies (m.suc.suc + i).divides((m + m.suc).binom(m))
    })
    primes_in_interval_divide(m.suc.suc, m, (m + m.suc).binom(m))
    interval_prime_product_divides(m.suc.suc, m, (m + m.suc).binom(m))
    range_prod(interval_prime_fn(m.suc.suc), m).divides((m + m.suc).binom(m))
}

/// The product of the primes above `m + 1` and at most `2m + 1` is at most four to the `m`.
///
/// The step a primorial induction turns on. The product divides the middle coefficient of the odd
/// row `2m + 1`, and that coefficient is at most `4^m`, so the product is too.
theorem odd_prime_interval_product_lte(m: Nat) {
    range_prod(interval_prime_fn(m.suc.suc), m) <= Nat.4.pow(m)
} by {
    odd_binom_prime_interval_product_divides(m)
    range_prod(interval_prime_fn(m.suc.suc), m).divides((m + m.suc).binom(m))
    binom_pos(m + m.suc, m)
    m <= m + m.suc
    Nat.0 < (m + m.suc).binom(m)
    (m + m.suc).binom(m) != Nat.0
    divides_lte(range_prod(interval_prime_fn(m.suc.suc), m), (m + m.suc).binom(m))
    ((m + m.suc).binom(m) = Nat.0
        or range_prod(interval_prime_fn(m.suc.suc), m) <= (m + m.suc).binom(m))
    range_prod(interval_prime_fn(m.suc.suc), m) <= (m + m.suc).binom(m)
    odd_binom_lte_four_pow(m)
    ((m + m.suc).binom(m) <= Nat.4.pow(m))
    lte_trans(range_prod(interval_prime_fn(m.suc.suc), m),
        (m + m.suc).binom(m), Nat.4.pow(m))
    range_prod(interval_prime_fn(m.suc.suc), m) <= Nat.4.pow(m)
}
