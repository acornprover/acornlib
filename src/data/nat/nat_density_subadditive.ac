from nat import Nat, from_nat
from real import Real, add_seq, lte_trans, mul_div_cancel, mul_left_cancel
from data.nat.nat_counting_function import count_upto, count_upto_zero, count_upto_suc_true,
    count_upto_suc_false
from data.nat.nat_density import range_size, range_size_positive, density_seq
from analysis import from_nat_real_lte, is_bounded_above, limsup, is_bounded_below,
    limsup_add, add_seq_bounded_above, add_seq_bounded_below
from data.nat.nat_density_mono import limsup_mono
from data.nat.nat_density_extremes import upper_density, density_seq_bounded_above,
    density_seq_bounded_below

numerals Nat

/// Transitivity of order on the naturals.
///
/// Stated locally because `lte_trans` on the naturals is shadowed by the one on the reals in any
/// module importing both, and this module needs both.
theorem nat_lte_trans(a: Nat, b: Nat, c: Nat) {
    a <= b and b <= c implies a <= c
} by {
    if a <= b and b <= c {
        ((a <= b) = exists(d: Nat) { a + d = b })
        let (d: Nat) satisfy {
            a + d = b
        }
        ((b <= c) = exists(e: Nat) { b + e = c })
        let (e: Nat) satisfy {
            b + e = c
        }
        (a + (d + e) = c)
        exists(f: Nat) { a + f = c }
        ((a <= c) = exists(f: Nat) { a + f = c })
        a <= c
    }
}

/// The disjunction of two predicates on the naturals.
define or_pred(p: Nat -> Bool, r: Nat -> Bool) -> (Nat -> Bool) {
    function(i: Nat) {
        p(i) or r(i)
    }
}

/// Membership in the disjunction is membership in one of the two.
theorem or_pred_apply(p: Nat -> Bool, r: Nat -> Bool, i: Nat) {
    or_pred(p, r)(i) = (p(i) or r(i))
} by {
    or_pred(p, r)(i) = (p(i) or r(i))
}

/// A disjunction is counted at most as often as the two parts together.
///
/// At each step at most one new member is added on the left, and it is matched by a new member
/// on the right unless neither predicate holds, in which case nothing changes.
theorem count_upto_or_le(p: Nat -> Bool, r: Nat -> Bool, n: Nat) {
    count_upto(or_pred(p, r), n) <= count_upto(p, n) + count_upto(r, n)
} by {
    define q(x: Nat) -> Bool {
        count_upto(or_pred(p, r), x) <= count_upto(p, x) + count_upto(r, x)
    }
    count_upto_zero(or_pred(p, r))
    count_upto_zero(p)
    count_upto_zero(r)
    (Nat.0 <= Nat.0 + Nat.0)
    q(Nat.0)
    forall(k: Nat) {
        if q(k) {
            count_upto(or_pred(p, r), k) <= count_upto(p, k) + count_upto(r, k)
            or_pred_apply(p, r, k)
            (or_pred(p, r)(k) = (p(k) or r(k)))
            if p(k) {
                or_pred(p, r)(k)
                count_upto_suc_true(or_pred(p, r), k)
                (count_upto(or_pred(p, r), k.suc) = count_upto(or_pred(p, r), k) + Nat.1)
                count_upto_suc_true(p, k)
                count_upto(p, k.suc) = count_upto(p, k) + Nat.1
                count_upto(r, k) <= count_upto(r, k.suc)
                (count_upto(or_pred(p, r), k) + Nat.1
                    <= (count_upto(p, k) + count_upto(r, k)) + Nat.1)
                ((count_upto(p, k) + count_upto(r, k)) + Nat.1
                    = (count_upto(p, k) + Nat.1) + count_upto(r, k))
                ((count_upto(p, k) + Nat.1) + count_upto(r, k)
                    <= (count_upto(p, k) + Nat.1) + count_upto(r, k.suc))
                nat_lte_trans(count_upto(or_pred(p, r), k) + Nat.1,
                    (count_upto(p, k) + Nat.1) + count_upto(r, k),
                    (count_upto(p, k) + Nat.1) + count_upto(r, k.suc))
                (count_upto(or_pred(p, r), k) + Nat.1
                    <= (count_upto(p, k) + Nat.1) + count_upto(r, k.suc))
                (count_upto(or_pred(p, r), k.suc)
                    <= count_upto(p, k.suc) + count_upto(r, k.suc))
            }
            if not p(k) {
                count_upto_suc_false(p, k)
                count_upto(p, k.suc) = count_upto(p, k)
                if r(k) {
                    or_pred(p, r)(k)
                    count_upto_suc_true(or_pred(p, r), k)
                    (count_upto(or_pred(p, r), k.suc)
                        = count_upto(or_pred(p, r), k) + Nat.1)
                    count_upto_suc_true(r, k)
                    count_upto(r, k.suc) = count_upto(r, k) + Nat.1
                    (count_upto(or_pred(p, r), k) + Nat.1
                        <= (count_upto(p, k) + count_upto(r, k)) + Nat.1)
                    ((count_upto(p, k) + count_upto(r, k)) + Nat.1
                        = count_upto(p, k) + (count_upto(r, k) + Nat.1))
                    (count_upto(or_pred(p, r), k.suc)
                        <= count_upto(p, k.suc) + count_upto(r, k.suc))
                }
                if not r(k) {
                    not or_pred(p, r)(k)
                    count_upto_suc_false(or_pred(p, r), k)
                    (count_upto(or_pred(p, r), k.suc) = count_upto(or_pred(p, r), k))
                    count_upto_suc_false(r, k)
                    count_upto(r, k.suc) = count_upto(r, k)
                    (count_upto(or_pred(p, r), k.suc)
                        <= count_upto(p, k.suc) + count_upto(r, k.suc))
                }
                (count_upto(or_pred(p, r), k.suc)
                    <= count_upto(p, k.suc) + count_upto(r, k.suc))
            }
            (count_upto(or_pred(p, r), k.suc)
                <= count_upto(p, k.suc) + count_upto(r, k.suc))
            q(k.suc)
        }
        (q(k) implies q(k.suc))
    }
    q(Nat.0) and forall(k: Nat) {
        q(k) implies q(k.suc)
    }
    Nat.induction(q)
    q(n)
    count_upto(or_pred(p, r), n) <= count_upto(p, n) + count_upto(r, n)
}

/// The density fraction of a disjunction is at most the sum of the two fractions.
theorem density_seq_or_le(p: Nat -> Bool, r: Nat -> Bool, n: Nat) {
    density_seq(or_pred(p, r), n) <= density_seq(p, n) + density_seq(r, n)
} by {
    count_upto_or_le(p, r, n.suc)
    (count_upto(or_pred(p, r), n.suc) <= count_upto(p, n.suc) + count_upto(r, n.suc))
    from_nat_real_lte(count_upto(or_pred(p, r), n.suc),
        count_upto(p, n.suc) + count_upto(r, n.suc))
    (from_nat[Real](count_upto(or_pred(p, r), n.suc))
        <= from_nat[Real](count_upto(p, n.suc) + count_upto(r, n.suc)))
    (from_nat[Real](count_upto(p, n.suc) + count_upto(r, n.suc))
        = from_nat[Real](count_upto(p, n.suc)) + from_nat[Real](count_upto(r, n.suc)))
    range_size_positive(n)
    Real.0 < range_size(n)
    (from_nat[Real](count_upto(or_pred(p, r), n.suc)) / range_size(n)
        <= (from_nat[Real](count_upto(p, n.suc))
            + from_nat[Real](count_upto(r, n.suc))) / range_size(n))
    range_size(n) != Real.0
    mul_div_cancel(from_nat[Real](count_upto(p, n.suc)), range_size(n))
    mul_div_cancel(from_nat[Real](count_upto(r, n.suc)), range_size(n))
    mul_div_cancel(from_nat[Real](count_upto(p, n.suc))
        + from_nat[Real](count_upto(r, n.suc)), range_size(n))
    (range_size(n) * (from_nat[Real](count_upto(p, n.suc)) / range_size(n)
        + from_nat[Real](count_upto(r, n.suc)) / range_size(n))
        = range_size(n) * (from_nat[Real](count_upto(p, n.suc)) / range_size(n))
            + range_size(n) * (from_nat[Real](count_upto(r, n.suc)) / range_size(n)))
    (range_size(n) * (from_nat[Real](count_upto(p, n.suc)) / range_size(n)
        + from_nat[Real](count_upto(r, n.suc)) / range_size(n))
        = from_nat[Real](count_upto(p, n.suc))
            + from_nat[Real](count_upto(r, n.suc)))
    (range_size(n) * ((from_nat[Real](count_upto(p, n.suc))
        + from_nat[Real](count_upto(r, n.suc))) / range_size(n))
        = from_nat[Real](count_upto(p, n.suc))
            + from_nat[Real](count_upto(r, n.suc)))
    mul_left_cancel(range_size(n),
        (from_nat[Real](count_upto(p, n.suc))
            + from_nat[Real](count_upto(r, n.suc))) / range_size(n),
        from_nat[Real](count_upto(p, n.suc)) / range_size(n)
            + from_nat[Real](count_upto(r, n.suc)) / range_size(n))
    ((from_nat[Real](count_upto(p, n.suc))
        + from_nat[Real](count_upto(r, n.suc))) / range_size(n)
        = from_nat[Real](count_upto(p, n.suc)) / range_size(n)
            + from_nat[Real](count_upto(r, n.suc)) / range_size(n))
    (density_seq(or_pred(p, r), n)
        = from_nat[Real](count_upto(or_pred(p, r), n.suc)) / range_size(n))
    (density_seq(p, n) = from_nat[Real](count_upto(p, n.suc)) / range_size(n))
    (density_seq(r, n) = from_nat[Real](count_upto(r, n.suc)) / range_size(n))
    density_seq(or_pred(p, r), n) <= density_seq(p, n) + density_seq(r, n)
}

/// The upper density is subadditive over a disjunction.
///
/// With monotonicity this is what bounds the density of a union of residue classes, and hence
/// what a covering system argument runs on.
theorem upper_density_subadditive(p: Nat -> Bool, r: Nat -> Bool) {
    upper_density(or_pred(p, r)) <= upper_density(p) + upper_density(r)
} by {
    density_seq_bounded_above(p)
    is_bounded_above(density_seq(p))
    density_seq_bounded_below(p)
    is_bounded_below(density_seq(p))
    density_seq_bounded_above(r)
    is_bounded_above(density_seq(r))
    density_seq_bounded_below(r)
    is_bounded_below(density_seq(r))
    density_seq_bounded_above(or_pred(p, r))
    is_bounded_above(density_seq(or_pred(p, r)))
    density_seq_bounded_below(or_pred(p, r))
    is_bounded_below(density_seq(or_pred(p, r)))
    forall(n: Nat) {
        density_seq_or_le(p, r, n)
        density_seq(or_pred(p, r), n) <= density_seq(p, n) + density_seq(r, n)
        (density_seq(or_pred(p, r))(n) = density_seq(or_pred(p, r), n))
        (density_seq(p)(n) = density_seq(p, n))
        (density_seq(r)(n) = density_seq(r, n))
        (add_seq(density_seq(p), density_seq(r))(n)
            = density_seq(p)(n) + density_seq(r)(n))
        (density_seq(or_pred(p, r))(n)
            <= add_seq(density_seq(p), density_seq(r))(n))
    }
    add_seq_bounded_above(density_seq(p), density_seq(r))
    is_bounded_above(add_seq(density_seq(p), density_seq(r)))
    add_seq_bounded_below(density_seq(p), density_seq(r))
    is_bounded_below(add_seq(density_seq(p), density_seq(r)))
    limsup_mono(density_seq(or_pred(p, r)), add_seq(density_seq(p), density_seq(r)))
    (limsup(density_seq(or_pred(p, r)))
        <= limsup(add_seq(density_seq(p), density_seq(r))))
    limsup_add(density_seq(p), density_seq(r))
    (limsup(add_seq(density_seq(p), density_seq(r)))
        <= limsup(density_seq(p)) + limsup(density_seq(r)))
    lte_trans(limsup(density_seq(or_pred(p, r))),
        limsup(add_seq(density_seq(p), density_seq(r))),
        limsup(density_seq(p)) + limsup(density_seq(r)))
    (limsup(density_seq(or_pred(p, r)))
        <= limsup(density_seq(p)) + limsup(density_seq(r)))
    (upper_density(or_pred(p, r)) = limsup(density_seq(or_pred(p, r))))
    (upper_density(p) = limsup(density_seq(p)))
    (upper_density(r) = limsup(density_seq(r)))
    upper_density(or_pred(p, r)) <= upper_density(p) + upper_density(r)
}
