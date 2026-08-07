from nat import Nat, from_nat
from list import List
from pair import Pair
from real import Real, lte_trans, mul_left_cancel, lte_add_left
from data.nat.nat_counting_function import none_pred
from data.nat.nat_density_extremes import upper_density, lower_density, upper_density_le_one,
    lower_density_le_upper_density, lower_density_nonneg
from data.nat.nat_residue_count import in_residue_class
from data.nat.nat_residue_density_value import residue_class_density, from_nat_modulus_pos
from data.nat.nat_density_subadditive import or_pred
from data.nat.nat_density_disjoint import disjoint_pred, lower_density_superadditive
from data.nat.nat_covering_density import covers_nat, covers_nat_cons, covers_nat_nil,
    system_density_real, system_density_real_cons, system_density_real_nil,
    system_reduced, system_reduced_head, system_reduced_tail, upper_density_none

numerals Nat

/// The lower density of a residue class in division form.
theorem lower_density_residue_class(m: Nat, a: Nat) {
    a < m implies lower_density(in_residue_class(m, a)) = Real.1 / from_nat[Real](m)
} by {
    if a < m {
        if m = Nat.0 {
            a < Nat.0
            not (a < Nat.0)
            false
        }
        m != Nat.0
        Nat.0 < m
        Nat.0.suc <= m
        (Nat.0.suc = Nat.1)
        Nat.1 <= m
        from_nat_modulus_pos(m)
        from_nat[Real](m) > Real.0
        from_nat[Real](m) != Real.0
        residue_class_density(m, a)
        (from_nat[Real](m) * lower_density(in_residue_class(m, a)) = Real.1)
        (Real.1 = from_nat[Real](m) * lower_density(in_residue_class(m, a)))
        mul_left_cancel(Real.1, from_nat[Real](m),
            lower_density(in_residue_class(m, a)))
        (Real.1 / from_nat[Real](m) = lower_density(in_residue_class(m, a)))
        lower_density(in_residue_class(m, a)) = Real.1 / from_nat[Real](m)
    }
}

/// True if the head class shares no natural with anything the rest of the system covers.
///
/// Stated directly rather than as a recursion over the list, which keeps it a plain definition
/// and is exactly the hypothesis superadditivity takes.
define head_disjoint_nat(head: Pair[Nat, Nat], rest: List[Pair[Nat, Nat]]) -> Bool {
    disjoint_pred(in_residue_class(head.first, head.second), covers_nat(rest))
}

/// One step of the disjoint bound.
///
/// If the head misses everything the rest covers, its weight adds to the lower density rather
/// than merely bounding it, which is what turns the subadditive bound around.
theorem disjoint_cons_bound(head: Pair[Nat, Nat], tail: List[Pair[Nat, Nat]]) {
    head.second < head.first and head_disjoint_nat(head, tail)
        and system_density_real(tail) <= lower_density(covers_nat(tail))
        implies system_density_real(List.cons(head, tail))
            <= lower_density(covers_nat(List.cons(head, tail)))
} by {
    if head.second < head.first and head_disjoint_nat(head, tail)
        and system_density_real(tail) <= lower_density(covers_nat(tail)) {
        (head_disjoint_nat(head, tail)
            = disjoint_pred(in_residue_class(head.first, head.second), covers_nat(tail)))
        disjoint_pred(in_residue_class(head.first, head.second), covers_nat(tail))
        lower_density_superadditive(
            in_residue_class(head.first, head.second), covers_nat(tail))
        (lower_density(in_residue_class(head.first, head.second))
            + lower_density(covers_nat(tail))
            <= lower_density(or_pred(in_residue_class(head.first, head.second),
                covers_nat(tail))))
        covers_nat_cons(head, tail)
        (covers_nat(List.cons(head, tail))
            = or_pred(in_residue_class(head.first, head.second), covers_nat(tail)))
        (lower_density(in_residue_class(head.first, head.second))
            + lower_density(covers_nat(tail))
            <= lower_density(covers_nat(List.cons(head, tail))))
        lower_density_residue_class(head.first, head.second)
        (lower_density(in_residue_class(head.first, head.second))
            = Real.1 / from_nat[Real](head.first))
        lte_add_left(system_density_real(tail), lower_density(covers_nat(tail)),
            Real.1 / from_nat[Real](head.first))
        (Real.1 / from_nat[Real](head.first) + system_density_real(tail)
            <= Real.1 / from_nat[Real](head.first) + lower_density(covers_nat(tail)))
        system_density_real_cons(head, tail)
        (system_density_real(List.cons(head, tail))
            = Real.1 / from_nat[Real](head.first) + system_density_real(tail))
        (system_density_real(List.cons(head, tail))
            <= Real.1 / from_nat[Real](head.first) + lower_density(covers_nat(tail)))
        (Real.1 / from_nat[Real](head.first) + lower_density(covers_nat(tail))
            <= lower_density(covers_nat(List.cons(head, tail))))
        lte_trans(system_density_real(List.cons(head, tail)),
            Real.1 / from_nat[Real](head.first) + lower_density(covers_nat(tail)),
            lower_density(covers_nat(List.cons(head, tail))))
        (system_density_real(List.cons(head, tail))
            <= lower_density(covers_nat(List.cons(head, tail))))
    }
}

/// True if the first list is a tail segment of the second.
define is_suffix_of(part: List[Pair[Nat, Nat]], whole: List[Pair[Nat, Nat]]) -> Bool {
    exists(front: List[Pair[Nat, Nat]]) {
        front + part = whole
    }
}

/// A list is a suffix of itself.
theorem is_suffix_of_refl(whole: List[Pair[Nat, Nat]]) {
    is_suffix_of(whole, whole)
} by {
    (List.nil[Pair[Nat, Nat]] + whole = whole)
    exists(front: List[Pair[Nat, Nat]]) { front + whole = whole }
    (is_suffix_of(whole, whole) = exists(front: List[Pair[Nat, Nat]]) {
        front + whole = whole
    })
    is_suffix_of(whole, whole)
}

/// A suffix of the tail is a suffix of the whole.
theorem is_suffix_of_cons(part: List[Pair[Nat, Nat]], head: Pair[Nat, Nat],
    tail: List[Pair[Nat, Nat]]) {
    is_suffix_of(part, tail) implies is_suffix_of(part, List.cons(head, tail))
} by {
    if is_suffix_of(part, tail) {
        (is_suffix_of(part, tail) = exists(front: List[Pair[Nat, Nat]]) {
            front + part = tail
        })
        let (front: List[Pair[Nat, Nat]]) satisfy {
            front + part = tail
        }
        (List.cons(head, front) + part = List.cons(head, front + part))
        (List.cons(head, front) + part = List.cons(head, tail))
        exists(f: List[Pair[Nat, Nat]]) { f + part = List.cons(head, tail) }
        (is_suffix_of(part, List.cons(head, tail))
            = exists(f: List[Pair[Nat, Nat]]) { f + part = List.cons(head, tail) })
        is_suffix_of(part, List.cons(head, tail))
    }
}

/// True if every class of the system misses everything the classes after it cover.
///
/// Written as a quantifier over the suffixes rather than as a recursion returning `Bool`. A
/// `Bool`-valued `match` whose cons case is a conjunction sends proof search into a shallow
/// explosion when unfolded, which is what `system_reduced` ran into.
define system_disjoint_nat(system: List[Pair[Nat, Nat]]) -> Bool {
    forall(head: Pair[Nat, Nat], tail: List[Pair[Nat, Nat]]) {
        is_suffix_of(List.cons(head, tail), system) implies head_disjoint_nat(head, tail)
    }
}

/// The head of a disjoint system misses what the tail covers.
theorem system_disjoint_nat_head(head: Pair[Nat, Nat], tail: List[Pair[Nat, Nat]]) {
    system_disjoint_nat(List.cons(head, tail)) implies head_disjoint_nat(head, tail)
} by {
    if system_disjoint_nat(List.cons(head, tail)) {
        (system_disjoint_nat(List.cons(head, tail))
            = forall(h: Pair[Nat, Nat], t: List[Pair[Nat, Nat]]) {
                is_suffix_of(List.cons(h, t), List.cons(head, tail))
                    implies head_disjoint_nat(h, t)
            })
        is_suffix_of_refl(List.cons(head, tail))
        is_suffix_of(List.cons(head, tail), List.cons(head, tail))
        (is_suffix_of(List.cons(head, tail), List.cons(head, tail))
            implies head_disjoint_nat(head, tail))
        head_disjoint_nat(head, tail)
    }
}

/// The tail of a disjoint system is disjoint.
theorem system_disjoint_nat_tail(head: Pair[Nat, Nat], tail: List[Pair[Nat, Nat]]) {
    system_disjoint_nat(List.cons(head, tail)) implies system_disjoint_nat(tail)
} by {
    if system_disjoint_nat(List.cons(head, tail)) {
        (system_disjoint_nat(List.cons(head, tail))
            = forall(h: Pair[Nat, Nat], t: List[Pair[Nat, Nat]]) {
                is_suffix_of(List.cons(h, t), List.cons(head, tail))
                    implies head_disjoint_nat(h, t)
            })
        forall(h: Pair[Nat, Nat], t: List[Pair[Nat, Nat]]) {
            if is_suffix_of(List.cons(h, t), tail) {
                is_suffix_of_cons(List.cons(h, t), head, tail)
                is_suffix_of(List.cons(h, t), List.cons(head, tail))
                (is_suffix_of(List.cons(h, t), List.cons(head, tail))
                    implies head_disjoint_nat(h, t))
                head_disjoint_nat(h, t)
            }
            (is_suffix_of(List.cons(h, t), tail) implies head_disjoint_nat(h, t))
        }
        (system_disjoint_nat(tail)
            = forall(h: Pair[Nat, Nat], t: List[Pair[Nat, Nat]]) {
                is_suffix_of(List.cons(h, t), tail) implies head_disjoint_nat(h, t)
            })
        system_disjoint_nat(tail)
    }
}

/// The weights of a reduced disjoint system are bounded by the lower density it covers.
theorem system_density_real_le_lower_density(system: List[Pair[Nat, Nat]]) {
    system_reduced(system) and system_disjoint_nat(system)
        implies system_density_real(system) <= lower_density(covers_nat(system))
} by {
    define p(s: List[Pair[Nat, Nat]]) -> Bool {
        system_reduced(s) and system_disjoint_nat(s)
            implies system_density_real(s) <= lower_density(covers_nat(s))
    }
    system_density_real_nil
    (system_density_real(List.nil[Pair[Nat, Nat]]) = Real.0)
    covers_nat_nil
    (covers_nat(List.nil[Pair[Nat, Nat]]) = none_pred)
    lower_density_nonneg(none_pred)
    Real.0 <= lower_density(none_pred)
    p(List.nil[Pair[Nat, Nat]])
    forall(head: Pair[Nat, Nat], tail: List[Pair[Nat, Nat]]) {
        if p(tail) {
            if system_reduced(List.cons(head, tail))
                and system_disjoint_nat(List.cons(head, tail)) {
                system_reduced_head(head, tail)
                head.second < head.first
                system_reduced_tail(head, tail)
                system_reduced(tail)
                system_disjoint_nat_head(head, tail)
                head_disjoint_nat(head, tail)
                system_disjoint_nat_tail(head, tail)
                system_disjoint_nat(tail)
                (system_reduced(tail) and system_disjoint_nat(tail)
                    implies system_density_real(tail) <= lower_density(covers_nat(tail)))
                system_density_real(tail) <= lower_density(covers_nat(tail))
                disjoint_cons_bound(head, tail)
                (system_density_real(List.cons(head, tail))
                    <= lower_density(covers_nat(List.cons(head, tail))))
            }
            p(List.cons(head, tail))
        }
        (p(tail) implies p(List.cons(head, tail)))
    }
    p(List.nil[Pair[Nat, Nat]]) and forall(h: Pair[Nat, Nat], t: List[Pair[Nat, Nat]]) {
        p(t) implies p(List.cons(h, t))
    }
    List.induction(p)
    p(system)
}

/// The reciprocals of the moduli of a disjoint system sum to at most one.
///
/// The classes are pairwise disjoint, so their weights add rather than merely bounding the
/// density of what they cover, and no density exceeds one. With the covering bound this pins
/// the density of a system that is both covering and disjoint at exactly one.
theorem disjoint_system_density_le_one(system: List[Pair[Nat, Nat]]) {
    system_reduced(system) and system_disjoint_nat(system)
        implies system_density_real(system) <= Real.1
} by {
    if system_reduced(system) and system_disjoint_nat(system) {
        system_density_real_le_lower_density(system)
        system_density_real(system) <= lower_density(covers_nat(system))
        lower_density_le_upper_density(covers_nat(system))
        lower_density(covers_nat(system)) <= upper_density(covers_nat(system))
        upper_density_le_one(covers_nat(system))
        upper_density(covers_nat(system)) <= Real.1
        lte_trans(lower_density(covers_nat(system)),
            upper_density(covers_nat(system)), Real.1)
        lower_density(covers_nat(system)) <= Real.1
        lte_trans(system_density_real(system),
            lower_density(covers_nat(system)), Real.1)
        system_density_real(system) <= Real.1
    }
}
