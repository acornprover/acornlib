from nat import Nat, add_sub, add_imp_sub_left, sub_self, lt_and_lte
from algebra.add_comm_monoid import AddCommMonoid
from algebra.add_semigroup import add_fn
from data.basic.functions import function_extensionality
from data.nat.nat_range_sum import range_sum, range_sum_zero, range_sum_suc, range_sum_one,
    range_sum_congr, range_sum_add

numerals Nat

/// The summand along the anti-diagonal of total `i`, at first coordinate `j`.
///
/// The pair is `(j, i - j)`, so the two coordinates add to `i` whenever `j <= i`. Above that the
/// truncation takes over, and nothing below reads the summand there.
define diag_fn[A: AddCommMonoid](h: Nat -> (Nat -> A), i: Nat, j: Nat) -> A {
    h(j)(i - j)
}

/// The sum along the anti-diagonal of total `i`.
define diag_sum[A: AddCommMonoid](h: Nat -> (Nat -> A), i: Nat) -> A {
    range_sum(diag_fn(h, i), i.suc)
}

/// The triangle of pairs with total at most `k`, summed by anti-diagonals.
define tri_diag_sum[A: AddCommMonoid](h: Nat -> (Nat -> A), k: Nat) -> A {
    range_sum(diag_sum(h), k.suc)
}

/// The part of row `a` inside the triangle of total at most `k`.
define row_upto_sum[A: AddCommMonoid](h: Nat -> (Nat -> A), k: Nat, a: Nat) -> A {
    range_sum(h(a), (k - a).suc)
}

/// The triangle of pairs with total at most `k`, summed by rows.
define tri_row_sum[A: AddCommMonoid](h: Nat -> (Nat -> A), k: Nat) -> A {
    range_sum(row_upto_sum(h, k), k.suc)
}

/// Raising the total by one lengthens each earlier row by exactly one term.
///
/// The new term of row `a` is the one whose coordinates add to `k + 1`, which is the summand of
/// the new anti-diagonal at `a`. That is what makes the two ways of enlarging the triangle agree.
theorem row_upto_sum_suc[A: AddCommMonoid](h: Nat -> (Nat -> A), k: Nat, a: Nat) {
    a <= k implies row_upto_sum(h, k.suc, a)
        = row_upto_sum(h, k, a) + diag_fn(h, k.suc, a)
} by {
    if a <= k {
        add_sub(k, a)
        ((k - a) + a = k)
        ((k - a).suc + a = k.suc)
        add_imp_sub_left(a, (k - a).suc, k.suc)
        (k.suc - a = (k - a).suc)
        (row_upto_sum(h, k.suc, a) = range_sum(h(a), (k.suc - a).suc))
        (row_upto_sum(h, k.suc, a) = range_sum(h(a), (k - a).suc.suc))
        range_sum_suc(h(a), (k - a).suc)
        (range_sum(h(a), (k - a).suc.suc)
            = range_sum(h(a), (k - a).suc) + h(a)((k - a).suc))
        (row_upto_sum(h, k, a) = range_sum(h(a), (k - a).suc))
        (diag_fn(h, k.suc, a) = h(a)(k.suc - a))
        (row_upto_sum(h, k.suc, a) = row_upto_sum(h, k, a) + diag_fn(h, k.suc, a))
    }
}

/// The last row of a triangle holds one term.
theorem row_upto_sum_top[A: AddCommMonoid](h: Nat -> (Nat -> A), k: Nat) {
    row_upto_sum(h, k, k) = h(k)(Nat.0)
} by {
    sub_self(k)
    (k - k = Nat.0)
    (row_upto_sum(h, k, k) = range_sum(h(k), Nat.0.suc))
    (Nat.0.suc = Nat.1)
    range_sum_one(h(k))
    (range_sum(h(k), Nat.1) = h(k)(Nat.0))
    row_upto_sum(h, k, k) = h(k)(Nat.0)
}

/// The last term of an anti-diagonal is the first entry of its last row.
theorem diag_fn_top[A: AddCommMonoid](h: Nat -> (Nat -> A), k: Nat) {
    diag_fn(h, k, k) = h(k)(Nat.0)
} by {
    sub_self(k)
    (k - k = Nat.0)
    diag_fn(h, k, k) = h(k)(Nat.0)
}

/// A triangle of pairs may be summed by anti-diagonals or by rows.
///
/// The reindexing a convolution rearrangement needs, and the one a rectangular interchange does
/// not give: the inner range depends on the outer index on both sides. Induction on the total,
/// where raising it by one adds the new anti-diagonal on one side and lengthens every row by its
/// term on the other.
theorem tri_sum_eq[A: AddCommMonoid](h: Nat -> (Nat -> A), k: Nat) {
    tri_diag_sum(h, k) = tri_row_sum(h, k)
} by {
    define p(m: Nat) -> Bool {
        tri_diag_sum(h, m) = tri_row_sum(h, m)
    }
    (tri_diag_sum(h, Nat.0) = range_sum(diag_sum(h), Nat.0.suc))
    (Nat.0.suc = Nat.1)
    range_sum_one(diag_sum(h))
    (range_sum(diag_sum(h), Nat.1) = diag_sum(h)(Nat.0))
    (diag_sum(h, Nat.0) = range_sum(diag_fn(h, Nat.0), Nat.1))
    range_sum_one(diag_fn(h, Nat.0))
    (range_sum(diag_fn(h, Nat.0), Nat.1) = diag_fn(h, Nat.0)(Nat.0))
    diag_fn_top(h, Nat.0)
    (diag_fn(h, Nat.0, Nat.0) = h(Nat.0)(Nat.0))
    (tri_row_sum(h, Nat.0) = range_sum(row_upto_sum(h, Nat.0), Nat.1))
    range_sum_one(row_upto_sum(h, Nat.0))
    (range_sum(row_upto_sum(h, Nat.0), Nat.1) = row_upto_sum(h, Nat.0)(Nat.0))
    row_upto_sum_top(h, Nat.0)
    (row_upto_sum(h, Nat.0, Nat.0) = h(Nat.0)(Nat.0))
    p(Nat.0)
    forall(m: Nat) {
        if p(m) {
            (tri_diag_sum(h, m) = tri_row_sum(h, m))
            range_sum_suc(diag_sum(h), m.suc)
            (tri_diag_sum(h, m.suc) = range_sum(diag_sum(h), m.suc.suc))
            (range_sum(diag_sum(h), m.suc.suc)
                = range_sum(diag_sum(h), m.suc) + diag_sum(h)(m.suc))
            (tri_diag_sum(h, m) = range_sum(diag_sum(h), m.suc))
            (diag_sum(h, m.suc) = range_sum(diag_fn(h, m.suc), m.suc.suc))
            range_sum_suc(diag_fn(h, m.suc), m.suc)
            (range_sum(diag_fn(h, m.suc), m.suc.suc)
                = range_sum(diag_fn(h, m.suc), m.suc) + diag_fn(h, m.suc)(m.suc))
            diag_fn_top(h, m.suc)
            (diag_fn(h, m.suc, m.suc) = h(m.suc)(Nat.0))
            (tri_diag_sum(h, m.suc) = tri_row_sum(h, m)
                + (range_sum(diag_fn(h, m.suc), m.suc) + h(m.suc)(Nat.0)))
            forall(a: Nat) {
                if a < m.suc {
                    a <= m
                    row_upto_sum_suc(h, m, a)
                    (row_upto_sum(h, m.suc, a)
                        = row_upto_sum(h, m, a) + diag_fn(h, m.suc, a))
                    (add_fn(row_upto_sum(h, m), diag_fn(h, m.suc))(a)
                        = row_upto_sum(h, m)(a) + diag_fn(h, m.suc)(a))
                    (row_upto_sum(h, m.suc)(a)
                        = add_fn(row_upto_sum(h, m), diag_fn(h, m.suc))(a))
                }
                (a < m.suc implies row_upto_sum(h, m.suc)(a)
                    = add_fn(row_upto_sum(h, m), diag_fn(h, m.suc))(a))
            }
            range_sum_congr(row_upto_sum(h, m.suc),
                add_fn(row_upto_sum(h, m), diag_fn(h, m.suc)), m.suc)
            (range_sum(row_upto_sum(h, m.suc), m.suc)
                = range_sum(add_fn(row_upto_sum(h, m), diag_fn(h, m.suc)), m.suc))
            range_sum_add(row_upto_sum(h, m), diag_fn(h, m.suc), m.suc)
            (range_sum(add_fn(row_upto_sum(h, m), diag_fn(h, m.suc)), m.suc)
                = range_sum(row_upto_sum(h, m), m.suc)
                    + range_sum(diag_fn(h, m.suc), m.suc))
            (tri_row_sum(h, m) = range_sum(row_upto_sum(h, m), m.suc))
            range_sum_suc(row_upto_sum(h, m.suc), m.suc)
            (tri_row_sum(h, m.suc) = range_sum(row_upto_sum(h, m.suc), m.suc.suc))
            (range_sum(row_upto_sum(h, m.suc), m.suc.suc)
                = range_sum(row_upto_sum(h, m.suc), m.suc)
                    + row_upto_sum(h, m.suc)(m.suc))
            row_upto_sum_top(h, m.suc)
            (row_upto_sum(h, m.suc, m.suc) = h(m.suc)(Nat.0))
            (tri_row_sum(h, m.suc) = (tri_row_sum(h, m)
                + range_sum(diag_fn(h, m.suc), m.suc)) + h(m.suc)(Nat.0))
            (tri_row_sum(h, m) + (range_sum(diag_fn(h, m.suc), m.suc) + h(m.suc)(Nat.0))
                = (tri_row_sum(h, m) + range_sum(diag_fn(h, m.suc), m.suc))
                    + h(m.suc)(Nat.0))
            tri_diag_sum(h, m.suc) = tri_row_sum(h, m.suc)
            p(m.suc)
        }
        (p(m) implies p(m.suc))
    }
    p(Nat.0) and forall(m: Nat) {
        p(m) implies p(m.suc)
    }
    Nat.induction(p)
    p(k)
}
