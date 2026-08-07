from nat import Nat, lte_trans, lte_and_lt, lt_and_lte, pow_add, pow_pow, pow_one,
    lt_mul_both, lte_mul_both
from list import partial
from combinatorics import binom, choose_symm_add_form
from data.nat.nat_range_sum import range_sum, range_sum_zero, range_sum_suc
from data.nat.nat_range_sum_bridge import range_sum_eq_partial
from data.nat.nat_binom_row_bound import range_sum_term_lte, binom_row, binom_row_sum

numerals Nat

/// Two distinct terms are together at most the whole sum, over the naturals.
///
/// The companion of `range_sum_term_lte`, and what a bound on one coefficient by half a row
/// needs. Splitting on whether the later index is the last one keeps the induction to two cases.
theorem range_sum_two_terms_lte(f: Nat -> Nat, n: Nat, j: Nat, k: Nat) {
    j < k and k < n implies f(j) + f(k) <= range_sum(f, n)
} by {
    define p(x: Nat) -> Bool {
        j < k and k < x implies f(j) + f(k) <= range_sum(f, x)
    }
    not (k < Nat.0)
    p(Nat.0)
    forall(i: Nat) {
        if p(i) {
            if j < k and k < i.suc {
                range_sum_suc(f, i)
                (range_sum(f, i.suc) = range_sum(f, i) + f(i))
                k <= i
                if k = i {
                    j < i
                    range_sum_term_lte(f, i, j)
                    (f(j) <= range_sum(f, i))
                    (f(j) + f(i) <= range_sum(f, i) + f(i))
                    f(j) + f(k) <= range_sum(f, i.suc)
                }
                if k != i {
                    k < i
                    (j < k and k < i implies f(j) + f(k) <= range_sum(f, i))
                    f(j) + f(k) <= range_sum(f, i)
                    (range_sum(f, i) <= range_sum(f, i) + f(i))
                    lte_trans(f(j) + f(k), range_sum(f, i), range_sum(f, i.suc))
                    f(j) + f(k) <= range_sum(f, i.suc)
                }
                f(j) + f(k) <= range_sum(f, i.suc)
            }
            p(i.suc)
        }
        (p(i) implies p(i.suc))
    }
    p(Nat.0) and forall(i: Nat) {
        p(i) implies p(i.suc)
    }
    Nat.induction(p)
    p(n)
}

/// Two distinct terms are together at most the whole partial sum.
theorem partial_two_terms_lte(f: Nat -> Nat, n: Nat, j: Nat, k: Nat) {
    j < k and k < n implies f(j) + f(k) <= partial(f, n)
} by {
    if j < k and k < n {
        range_sum_two_terms_lte(f, n, j, k)
        (f(j) + f(k) <= range_sum(f, n))
        range_sum_eq_partial(f, n)
        (range_sum(f, n) = partial(f, n))
        f(j) + f(k) <= partial(f, n)
    }
}

/// The middle two coefficients of an odd row are equal.
theorem odd_row_middle_symm(m: Nat) {
    (m + m.suc).binom(m) = (m + m.suc).binom(m.suc)
} by {
    choose_symm_add_form(m, m.suc)
    ((m + m.suc).binom(m) = (m + m.suc).binom(m.suc))
}

/// Twice the middle coefficient of an odd row is at most the row total.
theorem odd_row_middle_doubled(m: Nat) {
    (m + m.suc).binom(m) + (m + m.suc).binom(m) <= Nat.2.pow(m + m.suc)
} by {
    m < m.suc
    m.suc < (m + m.suc).suc
    partial_two_terms_lte(binom_row(m + m.suc), (m + m.suc).suc, m, m.suc)
    (binom_row(m + m.suc)(m) + binom_row(m + m.suc)(m.suc)
        <= partial(binom_row(m + m.suc), (m + m.suc).suc))
    (binom_row(m + m.suc, m) = (m + m.suc).binom(m))
    (binom_row(m + m.suc, m.suc) = (m + m.suc).binom(m.suc))
    odd_row_middle_symm(m)
    ((m + m.suc).binom(m) = (m + m.suc).binom(m.suc))
    binom_row_sum(m + m.suc)
    (partial(binom_row(m + m.suc), (m + m.suc).suc) = Nat.2.pow(m + m.suc))
    (m + m.suc).binom(m) + (m + m.suc).binom(m) <= Nat.2.pow(m + m.suc)
}

/// Two to an odd exponent is twice a power of four.
theorem two_pow_odd(m: Nat) {
    Nat.2.pow(m + m.suc) = Nat.2 * Nat.4.pow(m)
} by {
    (m + m.suc = (m + m) + Nat.1)
    pow_add[Nat](Nat.2, m + m, Nat.1)
    (Nat.2.pow((m + m) + Nat.1) = Nat.2.pow(m + m) * Nat.2.pow(Nat.1))
    pow_one[Nat](Nat.2)
    (Nat.2.pow(Nat.1) = Nat.2)
    (m + m = Nat.2 * m)
    pow_pow[Nat](Nat.2, Nat.2, m)
    (Nat.2.pow(Nat.2 * m) = Nat.2.pow(Nat.2).pow(m))
    (Nat.2.pow(Nat.2) = Nat.4)
    (Nat.2.pow(m + m) = Nat.4.pow(m))
    (Nat.4.pow(m) * Nat.2 = Nat.2 * Nat.4.pow(m))
    Nat.2.pow(m + m.suc) = Nat.2 * Nat.4.pow(m)
}

/// The middle coefficient of an odd row is at most four to the half index.
///
/// The bound the primorial argument uses: the two middle coefficients of row `2m + 1` are equal
/// and together no more than the row total, which is twice four to the `m`.
theorem odd_binom_lte_four_pow(m: Nat) {
    (m + m.suc).binom(m) <= Nat.4.pow(m)
} by {
    odd_row_middle_doubled(m)
    ((m + m.suc).binom(m) + (m + m.suc).binom(m) <= Nat.2.pow(m + m.suc))
    two_pow_odd(m)
    (Nat.2.pow(m + m.suc) = Nat.2 * Nat.4.pow(m))
    ((m + m.suc).binom(m) + (m + m.suc).binom(m)
        = Nat.2 * (m + m.suc).binom(m))
    (Nat.2 * (m + m.suc).binom(m) <= Nat.2 * Nat.4.pow(m))
    Nat.2 != Nat.0
    if Nat.4.pow(m) < (m + m.suc).binom(m) {
        lt_mul_both(Nat.2, Nat.4.pow(m), (m + m.suc).binom(m))
        (Nat.2 * Nat.4.pow(m) < Nat.2 * (m + m.suc).binom(m))
        false
    }
    (m + m.suc).binom(m) <= Nat.4.pow(m)
}
