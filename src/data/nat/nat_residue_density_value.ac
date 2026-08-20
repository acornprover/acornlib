from nat import Nat, from_nat
from real import Real, div_le_of_mul_le, mul_div_cancel, lte_trans, lte_lt_trans,
    lt_lte_trans, lte_antisymm, lte_mul_nonneg_left, pos_gt_zero, gt_zero_imp_pos,
    lte_add_right
from data.nat.nat_density import range_size, range_size_positive, density_seq
from analysis import from_nat_real_lte, lt_nat_multiple, tail_sup, tail_sup_is_least,
    is_bounded_above, limsup, limsup_le_tail_sup, is_bounded_below, liminf, tail_inf,
    tail_inf_is_greatest, liminf_ge_tail_inf, lte_of_lte_add_eps
from data.nat.nat_density_extremes import upper_density, lower_density, density_seq_bounded_above,
    density_seq_bounded_below, lower_density_le_upper_density
from data.nat.nat_residue_count import in_residue_class
from data.nat.nat_residue_density import lte_of_mul_lte_pos, residue_density_seq_upper,
    residue_density_seq_lower

/// A positive modulus embeds as a positive real.
theorem from_nat_modulus_pos(m: Nat) {
    Nat.1 <= m implies from_nat[Real](m) > Real.0
} by {
    if Nat.1 <= m {
        from_nat_real_lte(Nat.1, m)
        from_nat[Real](Nat.1) <= from_nat[Real](m)
        (from_nat[Real](Nat.1) = Real.1)
        Real.1 <= from_nat[Real](m)
        Real.0 < Real.1
        lt_lte_trans(Real.0, Real.1, from_nat[Real](m))
        Real.0 < from_nat[Real](m)
        from_nat[Real](m) > Real.0
    }
}

/// Beyond a threshold from the Archimedean property, every range is long enough.
theorem residue_range_long_enough(m: Nat, eps: Real, big_m: Nat, n: Nat) {
    eps > Real.0 and from_nat[Real](m) < eps * from_nat[Real](big_m) and big_m <= n
        implies from_nat[Real](m) <= eps * range_size(n)
} by {
    if eps > Real.0 and from_nat[Real](m) < eps * from_nat[Real](big_m) and big_m <= n {
        ((big_m <= n) = exists(c: Nat) { big_m + c = n })
        let (d: Nat) satisfy {
            big_m + d = n
        }
        (big_m + d.suc = n.suc)
        exists(e: Nat) { big_m + e = n.suc }
        ((big_m <= n.suc) = exists(e: Nat) { big_m + e = n.suc })
        big_m <= n.suc
        from_nat_real_lte(big_m, n.suc)
        from_nat[Real](big_m) <= from_nat[Real](n.suc)
        eps.is_positive
        not eps.is_negative
        lte_mul_nonneg_left(from_nat[Real](big_m), from_nat[Real](n.suc), eps)
        eps * from_nat[Real](big_m) <= eps * from_nat[Real](n.suc)
        lt_lte_trans(from_nat[Real](m), eps * from_nat[Real](big_m),
            eps * from_nat[Real](n.suc))
        from_nat[Real](m) < eps * from_nat[Real](n.suc)
        (range_size(n) = from_nat[Real](n.suc))
        from_nat[Real](m) <= eps * range_size(n)
    }
}

/// The modulus times the upper density of a residue class is at most one.
theorem residue_upper_density_le_one(m: Nat, a: Nat) {
    a < m implies from_nat[Real](m) * upper_density(in_residue_class(m, a)) <= Real.1
} by {
    if a < m {
        if m = Nat.0 {
            a < Nat.0
            not (a < Nat.0)
            false
        }
        m != Nat.0
        Nat.0 < m
        Nat.0.suc <= m
        (Nat.0.suc = Nat.1)
        Nat.1 <= m
        from_nat_modulus_pos(m)
        from_nat[Real](m) > Real.0
        density_seq_bounded_above(in_residue_class(m, a))
        is_bounded_above(density_seq(in_residue_class(m, a)))
        density_seq_bounded_below(in_residue_class(m, a))
        is_bounded_below(density_seq(in_residue_class(m, a)))
        forall(eps: Real) {
            if eps.is_positive {
                pos_gt_zero(eps)
                eps > Real.0
                lt_nat_multiple(from_nat[Real](m), eps)
                exists(j: Nat) { from_nat[Real](m) < eps * from_nat[Real](j) }
                let (big_m: Nat) satisfy {
                    from_nat[Real](m) < eps * from_nat[Real](big_m)
                }
                forall(n: Nat) {
                    if big_m <= n {
                        residue_range_long_enough(m, eps, big_m, n)
                        from_nat[Real](m) <= eps * range_size(n)
                        residue_density_seq_upper(m, a, eps, n)
                        (from_nat[Real](m) * density_seq(in_residue_class(m, a), n)
                            <= Real.1 + eps)
                        (density_seq(in_residue_class(m, a), n) * from_nat[Real](m)
                            <= Real.1 + eps)
                        div_le_of_mul_le(density_seq(in_residue_class(m, a), n),
                            from_nat[Real](m), Real.1 + eps)
                        (density_seq(in_residue_class(m, a), n)
                            <= (Real.1 + eps) / from_nat[Real](m))
                        (density_seq(in_residue_class(m, a))(n)
                            = density_seq(in_residue_class(m, a), n))
                        (density_seq(in_residue_class(m, a))(n)
                            <= (Real.1 + eps) / from_nat[Real](m))
                    }
                    (big_m <= n implies density_seq(in_residue_class(m, a))(n)
                        <= (Real.1 + eps) / from_nat[Real](m))
                }
                tail_sup_is_least(density_seq(in_residue_class(m, a)), big_m,
                    (Real.1 + eps) / from_nat[Real](m))
                (tail_sup(density_seq(in_residue_class(m, a)), big_m)
                    <= (Real.1 + eps) / from_nat[Real](m))
                limsup_le_tail_sup(density_seq(in_residue_class(m, a)), big_m)
                (limsup(density_seq(in_residue_class(m, a)))
                    <= tail_sup(density_seq(in_residue_class(m, a)), big_m))
                lte_trans(limsup(density_seq(in_residue_class(m, a))),
                    tail_sup(density_seq(in_residue_class(m, a)), big_m),
                    (Real.1 + eps) / from_nat[Real](m))
                (limsup(density_seq(in_residue_class(m, a)))
                    <= (Real.1 + eps) / from_nat[Real](m))
                (upper_density(in_residue_class(m, a))
                    = limsup(density_seq(in_residue_class(m, a))))
                not from_nat[Real](m).is_negative
                lte_mul_nonneg_left(upper_density(in_residue_class(m, a)),
                    (Real.1 + eps) / from_nat[Real](m), from_nat[Real](m))
                (from_nat[Real](m) * upper_density(in_residue_class(m, a))
                    <= from_nat[Real](m) * ((Real.1 + eps) / from_nat[Real](m)))
                from_nat[Real](m) != Real.0
                mul_div_cancel(Real.1 + eps, from_nat[Real](m))
                (from_nat[Real](m) * ((Real.1 + eps) / from_nat[Real](m)) = Real.1 + eps)
                (from_nat[Real](m) * upper_density(in_residue_class(m, a))
                    <= Real.1 + eps)
            }
            (eps.is_positive implies from_nat[Real](m)
                * upper_density(in_residue_class(m, a)) <= Real.1 + eps)
        }
        lte_of_lte_add_eps(from_nat[Real](m) * upper_density(in_residue_class(m, a)),
            Real.1)
        from_nat[Real](m) * upper_density(in_residue_class(m, a)) <= Real.1
    }
}

/// The modulus times the lower density of a residue class is at least one.
theorem residue_lower_density_ge_one(m: Nat, a: Nat) {
    a < m implies Real.1 <= from_nat[Real](m) * lower_density(in_residue_class(m, a))
} by {
    if a < m {
        if m = Nat.0 {
            a < Nat.0
            not (a < Nat.0)
            false
        }
        m != Nat.0
        Nat.0 < m
        Nat.0.suc <= m
        (Nat.0.suc = Nat.1)
        Nat.1 <= m
        from_nat_modulus_pos(m)
        from_nat[Real](m) > Real.0
        from_nat[Real](m) != Real.0
        density_seq_bounded_above(in_residue_class(m, a))
        is_bounded_above(density_seq(in_residue_class(m, a)))
        density_seq_bounded_below(in_residue_class(m, a))
        is_bounded_below(density_seq(in_residue_class(m, a)))
        forall(eps: Real) {
            if eps.is_positive {
                pos_gt_zero(eps)
                eps > Real.0
                lt_nat_multiple(from_nat[Real](m), eps)
                exists(j: Nat) { from_nat[Real](m) < eps * from_nat[Real](j) }
                let (big_m: Nat) satisfy {
                    from_nat[Real](m) < eps * from_nat[Real](big_m)
                }
                forall(n: Nat) {
                    if big_m <= n {
                        residue_range_long_enough(m, eps, big_m, n)
                        from_nat[Real](m) <= eps * range_size(n)
                        residue_density_seq_lower(m, a, eps, n)
                        (Real.1 - eps
                            <= from_nat[Real](m) * density_seq(in_residue_class(m, a), n))
                        mul_div_cancel(Real.1 - eps, from_nat[Real](m))
                        (from_nat[Real](m) * ((Real.1 - eps) / from_nat[Real](m))
                            = Real.1 - eps)
                        (((Real.1 - eps) / from_nat[Real](m)) * from_nat[Real](m)
                            <= density_seq(in_residue_class(m, a), n) * from_nat[Real](m))
                        lte_of_mul_lte_pos((Real.1 - eps) / from_nat[Real](m),
                            density_seq(in_residue_class(m, a), n), from_nat[Real](m))
                        ((Real.1 - eps) / from_nat[Real](m)
                            <= density_seq(in_residue_class(m, a), n))
                        (density_seq(in_residue_class(m, a))(n)
                            = density_seq(in_residue_class(m, a), n))
                        ((Real.1 - eps) / from_nat[Real](m)
                            <= density_seq(in_residue_class(m, a))(n))
                    }
                    (big_m <= n implies (Real.1 - eps) / from_nat[Real](m)
                        <= density_seq(in_residue_class(m, a))(n))
                }
                tail_inf_is_greatest(density_seq(in_residue_class(m, a)), big_m,
                    (Real.1 - eps) / from_nat[Real](m))
                ((Real.1 - eps) / from_nat[Real](m)
                    <= tail_inf(density_seq(in_residue_class(m, a)), big_m))
                liminf_ge_tail_inf(density_seq(in_residue_class(m, a)), big_m)
                (tail_inf(density_seq(in_residue_class(m, a)), big_m)
                    <= liminf(density_seq(in_residue_class(m, a))))
                lte_trans((Real.1 - eps) / from_nat[Real](m),
                    tail_inf(density_seq(in_residue_class(m, a)), big_m),
                    liminf(density_seq(in_residue_class(m, a))))
                ((Real.1 - eps) / from_nat[Real](m)
                    <= liminf(density_seq(in_residue_class(m, a))))
                (lower_density(in_residue_class(m, a))
                    = liminf(density_seq(in_residue_class(m, a))))
                not from_nat[Real](m).is_negative
                lte_mul_nonneg_left((Real.1 - eps) / from_nat[Real](m),
                    lower_density(in_residue_class(m, a)), from_nat[Real](m))
                (from_nat[Real](m) * ((Real.1 - eps) / from_nat[Real](m))
                    <= from_nat[Real](m) * lower_density(in_residue_class(m, a)))
                mul_div_cancel(Real.1 - eps, from_nat[Real](m))
                (from_nat[Real](m) * ((Real.1 - eps) / from_nat[Real](m))
                    = Real.1 - eps)
                (Real.1 - eps
                    <= from_nat[Real](m) * lower_density(in_residue_class(m, a)))
                lte_add_right(Real.1 - eps,
                    from_nat[Real](m) * lower_density(in_residue_class(m, a)), eps)
                ((Real.1 - eps) + eps
                    <= (from_nat[Real](m) * lower_density(in_residue_class(m, a))) + eps)
                ((Real.1 - eps) + eps = Real.1)
                (Real.1
                    <= (from_nat[Real](m) * lower_density(in_residue_class(m, a))) + eps)
            }
            (eps.is_positive implies Real.1
                <= (from_nat[Real](m) * lower_density(in_residue_class(m, a))) + eps)
        }
        lte_of_lte_add_eps(Real.1,
            from_nat[Real](m) * lower_density(in_residue_class(m, a)))
        Real.1 <= from_nat[Real](m) * lower_density(in_residue_class(m, a))
    }
}

/// A residue class has density the reciprocal of its modulus.
///
/// Stated cross-multiplied, so no division appears: the modulus times either one-sided density
/// is one. Both always exist, unlike the natural density, so this is the strongest form
/// reachable without building a `converges_to`.
theorem residue_class_density(m: Nat, a: Nat) {
    a < m implies from_nat[Real](m) * lower_density(in_residue_class(m, a)) = Real.1
        and from_nat[Real](m) * upper_density(in_residue_class(m, a)) = Real.1
} by {
    if a < m {
        if m = Nat.0 {
            a < Nat.0
            not (a < Nat.0)
            false
        }
        m != Nat.0
        Nat.0 < m
        Nat.0.suc <= m
        (Nat.0.suc = Nat.1)
        Nat.1 <= m
        from_nat_modulus_pos(m)
        from_nat[Real](m) > Real.0
        not from_nat[Real](m).is_negative
        residue_lower_density_ge_one(m, a)
        Real.1 <= from_nat[Real](m) * lower_density(in_residue_class(m, a))
        residue_upper_density_le_one(m, a)
        from_nat[Real](m) * upper_density(in_residue_class(m, a)) <= Real.1
        lower_density_le_upper_density(in_residue_class(m, a))
        (lower_density(in_residue_class(m, a)) <= upper_density(in_residue_class(m, a)))
        lte_mul_nonneg_left(lower_density(in_residue_class(m, a)),
            upper_density(in_residue_class(m, a)), from_nat[Real](m))
        (from_nat[Real](m) * lower_density(in_residue_class(m, a))
            <= from_nat[Real](m) * upper_density(in_residue_class(m, a)))
        lte_trans(from_nat[Real](m) * lower_density(in_residue_class(m, a)),
            from_nat[Real](m) * upper_density(in_residue_class(m, a)), Real.1)
        (from_nat[Real](m) * lower_density(in_residue_class(m, a)) <= Real.1)
        lte_antisymm(from_nat[Real](m) * lower_density(in_residue_class(m, a)), Real.1)
        from_nat[Real](m) * lower_density(in_residue_class(m, a)) = Real.1
        lte_trans(Real.1, from_nat[Real](m) * lower_density(in_residue_class(m, a)),
            from_nat[Real](m) * upper_density(in_residue_class(m, a)))
        (Real.1 <= from_nat[Real](m) * upper_density(in_residue_class(m, a)))
        lte_antisymm(from_nat[Real](m) * upper_density(in_residue_class(m, a)), Real.1)
        from_nat[Real](m) * upper_density(in_residue_class(m, a)) = Real.1
        (from_nat[Real](m) * lower_density(in_residue_class(m, a)) = Real.1
            and from_nat[Real](m) * upper_density(in_residue_class(m, a)) = Real.1)
    }
}
