from nat import Nat, divides_lte, divides_self, factorial_step, factorial_one, lte_and_lt,
    lt_and_lte, lte_trans, add_imp_sub, lte_add_left, lt_add_left, lte_add_right
from number_theory import central_binom, central_binom_eq, falling_product,
    falling_product_eq_binom_mul_factorial, prime_divides_mul
from falling_product_prime_factors import prime_factor_divides_falling_product

numerals Nat

/// A prime dividing a factorial is at most the argument.
///
/// The factorial is built one factor at a time, and Euclid's lemma sends the prime into one of
/// them. Nothing in `src/number_theory/` bounds a prime this way, though it is what separates
/// the two halves of a binomial-times-factorial product.
theorem prime_divides_factorial_imp_lte(p: Nat, n: Nat) {
    p.is_prime and p.divides(n.factorial) implies p <= n
} by {
    if p.is_prime {
        define q(x: Nat) -> Bool {
            p.divides(x.factorial) implies p <= x
        }
        if p.divides(Nat.0.factorial) {
            factorial_one
            Nat.0.factorial = Nat.1
            p.divides(Nat.1)
            divides_lte(p, Nat.1)
            (Nat.1 = Nat.0 or p <= Nat.1)
            Nat.0 < Nat.1
            Nat.1 != Nat.0
            p <= Nat.1
            (p.is_prime = (Nat.1 < p and not p.is_composite))
            Nat.1 < p
            lte_and_lt(p, Nat.1, p)
            p < p
            false
        }
        if false {
            Nat.2 <= p
            false
        }
        q(Nat.0)
        forall(m: Nat) {
            if q(m) {
                if p.divides(m.suc.factorial) {
                    factorial_step(m)
                    m.suc.factorial = m.suc * m.factorial
                    p.divides(m.suc * m.factorial)
                    prime_divides_mul(p, m.suc, m.factorial)
                    (p.divides(m.suc) or p.divides(m.factorial))
                    if p.divides(m.suc) {
                        divides_lte(p, m.suc)
                        (m.suc = Nat.0 or p <= m.suc)
                        Nat.0 < m.suc
                        m.suc != Nat.0
                        p <= m.suc
                    }
                    if not p.divides(m.suc) {
                        p.divides(m.factorial)
                        (p.divides(m.factorial) implies p <= m)
                        p <= m
                        m <= m.suc
                        lte_trans(p, m, m.suc)
                        p <= m.suc
                    }
                    p <= m.suc
                }
                q(m.suc)
            }
            (q(m) implies q(m.suc))
        }
        q(Nat.0) and forall(m: Nat) {
            q(m) implies q(m.suc)
        }
        Nat.induction(q)
        q(n)
        (p.divides(n.factorial) implies p <= n)
    }
}

/// A prime above `n` and at most `2n` divides the central binomial coefficient.
///
/// The falling product `2n * (2n - 1) * ... * (n + 1)` has such a prime among its factors, and
/// that product is the central binomial coefficient times `n!`. The prime cannot come from the
/// factorial, since a prime dividing `n!` is at most `n`, so it comes from the binomial
/// coefficient.
///
/// This is the divisibility statement that a proof of Bertrand's postulate starts from: it is
/// what lets the product of the primes in `(n, 2n]` be bounded by the central binomial
/// coefficient.
theorem prime_in_interval_divides_central_binom(p: Nat, m: Nat) {
    p.is_prime and m.suc < p and p <= m.suc + m.suc
        implies p.divides(central_binom(m.suc))
} by {
    if p.is_prime and m.suc < p and p <= m.suc + m.suc {
        (p <= m.suc + m.suc) = exists(c: Nat) { p + c = m.suc + m.suc }
        let (i: Nat) satisfy {
            p + i = m.suc + m.suc
        }
        if m.suc <= i {
            lte_add_left(m.suc, m.suc, i)
            m.suc + m.suc <= m.suc + i
            lt_add_left(i, m.suc, p)
            i + m.suc < i + p
            m.suc + i < p + i
            lte_and_lt(m.suc + m.suc, m.suc + i, p + i)
            m.suc + m.suc < p + i
            m.suc + m.suc < m.suc + m.suc
            false
        }
        not (m.suc <= i)
        i < m.suc
        i <= m
        i + p = m.suc + m.suc
        add_imp_sub(p, i, m.suc + m.suc)
        (m.suc + m.suc) - i = p
        divides_self(p)
        p.divides((m.suc + m.suc) - i)
        prime_factor_divides_falling_product(m.suc + m.suc, m, p, i)
        p.divides(falling_product(m.suc + m.suc, m))
        m < m.suc
        m.suc <= m.suc + m.suc
        lt_and_lte(m, m.suc, m.suc + m.suc)
        m < m.suc + m.suc
        falling_product_eq_binom_mul_factorial(m.suc + m.suc, m)
        (falling_product(m.suc + m.suc, m)
            = (m.suc + m.suc).binom(m.suc) * m.suc.factorial)
        p.divides((m.suc + m.suc).binom(m.suc) * m.suc.factorial)
        prime_divides_mul(p, (m.suc + m.suc).binom(m.suc), m.suc.factorial)
        (p.divides((m.suc + m.suc).binom(m.suc)) or p.divides(m.suc.factorial))
        if p.divides(m.suc.factorial) {
            prime_divides_factorial_imp_lte(p, m.suc)
            p <= m.suc
            lte_and_lt(p, m.suc, p)
            p < p
            false
        }
        p.divides((m.suc + m.suc).binom(m.suc))
        central_binom_eq(m.suc)
        central_binom(m.suc) = (m.suc + m.suc).binom(m.suc)
        p.divides(central_binom(m.suc))
    }
}
