from nat import Nat, mul_assoc, mul_comm, divides_symm
from data.nat.nat_square import is_square, is_square_intro, is_square_witness

numerals Nat

/// True if no square other than one divides `n`.
///
/// Stated through divisibility rather than through valuations, so it needs no
/// factorisation machinery and applies directly wherever a divisor is in hand.
define is_squarefree(n: Nat) -> Bool {
    forall(d: Nat) {
        is_square(d) and d.divides(n) implies d = Nat.1
    }
}

/// A square divisor of a squarefree number is one.
theorem is_squarefree_apply(n: Nat, d: Nat) {
    is_squarefree(n) and is_square(d) and d.divides(n) implies d = Nat.1
} by {
    if is_squarefree(n) and is_square(d) and d.divides(n) {
        is_squarefree(n) = forall(e: Nat) {
            is_square(e) and e.divides(n) implies e = Nat.1
        }
        forall(e: Nat) {
            is_square(e) and e.divides(n) implies e = Nat.1
        }
        is_square(d) and d.divides(n) implies d = Nat.1
        d = Nat.1
    }
}

/// A pointwise condition on square divisors is squarefreeness.
theorem is_squarefree_intro(n: Nat) {
    (forall(d: Nat) { is_square(d) and d.divides(n) implies d = Nat.1 }) implies is_squarefree(n)
} by {
    if forall(d: Nat) { is_square(d) and d.divides(n) implies d = Nat.1 } {
        is_squarefree(n) = forall(e: Nat) {
            is_square(e) and e.divides(n) implies e = Nat.1
        }
        is_squarefree(n)
    }
}

/// One is squarefree.
theorem one_is_squarefree {
    is_squarefree(Nat.1)
} by {
    forall(d: Nat) {
        if is_square(d) and d.divides(Nat.1) {
            d = Nat.1 * d
            Nat.1.divides(d)
            divides_symm(d, Nat.1)
            d = Nat.1
        }
        is_square(d) and d.divides(Nat.1) implies d = Nat.1
    }
    is_squarefree_intro(Nat.1)
    is_squarefree(Nat.1)
}

/// A squarefree number that is a square is one.
///
/// The two predicates meet only at one, since a number always divides itself.
theorem squarefree_and_square_is_one(n: Nat) {
    is_squarefree(n) and is_square(n) implies n = Nat.1
} by {
    if is_squarefree(n) and is_square(n) {
        n.divides(n)
        is_squarefree_apply(n, n)
        n = Nat.1
    }
}

/// Squarefreeness passes to divisors.
///
/// A square dividing a divisor divides the original, so it is one.
theorem squarefree_of_divides(n: Nat, m: Nat) {
    is_squarefree(n) and m.divides(n) implies is_squarefree(m)
} by {
    if is_squarefree(n) and m.divides(n) {
        forall(d: Nat) {
            if is_square(d) and d.divides(m) {
                d.divides(n)
                is_squarefree_apply(n, d)
                d = Nat.1
            }
            is_square(d) and d.divides(m) implies d = Nat.1
        }
        is_squarefree_intro(m)
        is_squarefree(m)
    }
}

/// A number divisible by a square other than one is not squarefree.
theorem not_squarefree_of_square_divisor(n: Nat, d: Nat) {
    is_square(d) and d.divides(n) and d != Nat.1 implies not is_squarefree(n)
} by {
    if is_square(d) and d.divides(n) and d != Nat.1 {
        if is_squarefree(n) {
            is_squarefree_apply(n, d)
            d = Nat.1
            false
        }
        not is_squarefree(n)
    }
}
