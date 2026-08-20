from nat import Nat, from_nat
from real import Real, div_le_of_mul_le, lte_trans, lte_lt_trans, lt_lte_trans,
    lte_add_right, lte_mul_nonneg_left, lte_antisymm, pos_gt_zero
from data.nat.nat_counting_function import count_upto, count_upto_zero, count_upto_suc_true,
    count_upto_suc_false
from data.nat.nat_density import range_size, range_size_positive, density_seq, eventually_all,
    eventually_all_witness, eventually_all_of_all, eventually_all_and, and_pred
from analysis import from_nat_real_lte, lt_nat_multiple, is_bounded_above,
    is_bounded_below, liminf, tail_inf, tail_inf_is_greatest, liminf_ge_tail_inf,
    lte_of_lte_add_eps
from data.nat.nat_density_extremes import upper_density, lower_density, density_seq_bounded_above,
    density_seq_bounded_below, upper_density_le_one, lower_density_le_upper_density

/// Transitivity of order on the naturals.
///
/// Stated locally because `lte_trans` on the naturals is shadowed by the one on the reals in
/// any module importing both, and this module needs both.
theorem nat_lte_trans(a: Nat, b: Nat, c: Nat) {
    a <= b and b <= c implies a <= c
} by {
    if a <= b and b <= c {
        ((a <= b) = exists(d: Nat) { a + d = b })
        let (d: Nat) satisfy {
            a + d = b
        }
        ((b <= c) = exists(e: Nat) { b + e = c })
        let (e: Nat) satisfy {
            b + e = c
        }
        (a + (d + e) = c)
        exists(f: Nat) { a + f = c }
        ((a <= c) = exists(f: Nat) { a + f = c })
        a <= c
    }
}

/// A predicate holding from a point on is missed at most that many times.
///
/// Written as `n <= count + threshold` rather than with a subtraction, so that the induction
/// step needs no case analysis on which side is larger.
theorem count_upto_of_holds_from(p: Nat -> Bool, big_n: Nat, n: Nat) {
    (forall(m: Nat) { big_n <= m implies p(m) })
        implies n <= count_upto(p, n) + big_n
} by {
    if forall(m: Nat) { big_n <= m implies p(m) } {
        define q(x: Nat) -> Bool {
            x <= count_upto(p, x) + big_n
        }
        count_upto_zero(p)
        count_upto(p, Nat.0) = Nat.0
        Nat.0 <= Nat.0 + big_n
        q(Nat.0)
        forall(k: Nat) {
            if q(k) {
                k <= count_upto(p, k) + big_n
                if p(k) {
                    count_upto_suc_true(p, k)
                    count_upto(p, k.suc) = count_upto(p, k) + Nat.1
                    (k + Nat.1 = k.suc)
                    ((count_upto(p, k) + big_n) + Nat.1
                        = (count_upto(p, k) + Nat.1) + big_n)
                    k + Nat.1 <= (count_upto(p, k) + big_n) + Nat.1
                    k.suc <= (count_upto(p, k) + Nat.1) + big_n
                    k.suc <= count_upto(p, k.suc) + big_n
                }
                if not p(k) {
                    if big_n <= k {
                        (big_n <= k implies p(k))
                        p(k)
                        false
                    }
                    not (big_n <= k)
                    k < big_n
                    k.suc <= big_n
                    big_n <= count_upto(p, k.suc) + big_n
                    nat_lte_trans(k.suc, big_n, count_upto(p, k.suc) + big_n)
                    k.suc <= count_upto(p, k.suc) + big_n
                }
                k.suc <= count_upto(p, k.suc) + big_n
                q(k.suc)
            }
            (q(k) implies q(k.suc))
        }
        q(Nat.0) and forall(k: Nat) {
            q(k) implies q(k.suc)
        }
        Nat.induction(q)
        q(n)
        n <= count_upto(p, n) + big_n
    }
}

/// The density fraction of a predicate holding from a point on, bounded below.
///
/// Once the range is long enough that the threshold is a small fraction of it, the fraction of
/// members satisfying the predicate is within that fraction of one.
theorem density_seq_ge_of_holds_from(
    p: Nat -> Bool, big_n: Nat, eps: Real, k: Nat
) {
    (forall(m: Nat) { big_n <= m implies p(m) }) and eps > Real.0
        and from_nat[Real](big_n) <= eps * range_size(k)
        implies Real.1 - eps <= density_seq(p, k)
} by {
    if (forall(m: Nat) { big_n <= m implies p(m) }) and eps > Real.0
        and from_nat[Real](big_n) <= eps * range_size(k) {
        count_upto_of_holds_from(p, big_n, k.suc)
        k.suc <= count_upto(p, k.suc) + big_n
        from_nat_real_lte(k.suc, count_upto(p, k.suc) + big_n)
        (from_nat[Real](k.suc) <= from_nat[Real](count_upto(p, k.suc) + big_n))
        (from_nat[Real](count_upto(p, k.suc) + big_n)
            = from_nat[Real](count_upto(p, k.suc)) + from_nat[Real](big_n))
        (from_nat[Real](k.suc)
            <= from_nat[Real](count_upto(p, k.suc)) + from_nat[Real](big_n))
        (range_size(k) = from_nat[Real](k.suc))
        (range_size(k) <= from_nat[Real](count_upto(p, k.suc)) + from_nat[Real](big_n))
        (from_nat[Real](big_n) <= eps * range_size(k))
        (from_nat[Real](count_upto(p, k.suc)) + from_nat[Real](big_n)
            <= from_nat[Real](count_upto(p, k.suc)) + eps * range_size(k))
        lte_trans(range_size(k),
            from_nat[Real](count_upto(p, k.suc)) + from_nat[Real](big_n),
            from_nat[Real](count_upto(p, k.suc)) + eps * range_size(k))
        (range_size(k)
            <= from_nat[Real](count_upto(p, k.suc)) + eps * range_size(k))
        ((Real.1 - eps) * range_size(k) = range_size(k) - eps * range_size(k))
        lte_add_right(range_size(k),
            from_nat[Real](count_upto(p, k.suc)) + eps * range_size(k),
            -(eps * range_size(k)))
        (range_size(k) + -(eps * range_size(k))
            <= (from_nat[Real](count_upto(p, k.suc)) + eps * range_size(k))
                + -(eps * range_size(k)))
        (range_size(k) + -(eps * range_size(k))
            = range_size(k) - eps * range_size(k))
        ((from_nat[Real](count_upto(p, k.suc)) + eps * range_size(k))
            + -(eps * range_size(k))
            = from_nat[Real](count_upto(p, k.suc))
                + (eps * range_size(k) + -(eps * range_size(k))))
        (eps * range_size(k) + -(eps * range_size(k)) = Real.0)
        (from_nat[Real](count_upto(p, k.suc)) + Real.0
            = from_nat[Real](count_upto(p, k.suc)))
        ((from_nat[Real](count_upto(p, k.suc)) + eps * range_size(k))
            + -(eps * range_size(k)) = from_nat[Real](count_upto(p, k.suc)))
        (range_size(k) - eps * range_size(k)
            <= from_nat[Real](count_upto(p, k.suc)))
        ((Real.1 - eps) * range_size(k) <= from_nat[Real](count_upto(p, k.suc)))
        range_size_positive(k)
        Real.0 < range_size(k)
        range_size(k) > Real.0
        div_le_of_mul_le(Real.1 - eps, range_size(k),
            from_nat[Real](count_upto(p, k.suc)))
        (Real.1 - eps <= from_nat[Real](count_upto(p, k.suc)) / range_size(k))
        (density_seq(p, k) = from_nat[Real](count_upto(p, k.suc)) / range_size(k))
        Real.1 - eps <= density_seq(p, k)
    }
}

/// The lower density of a predicate holding from some point on is one.
///
/// For each size, the Archimedean property gives a range long enough that the threshold is
/// below that fraction of it; from there on the density fraction is within the size of one, so
/// the tail infimum is, and so is the limit inferior.
theorem lower_density_of_eventually_all(p: Nat -> Bool) {
    eventually_all(p) implies lower_density(p) = Real.1
} by {
    if eventually_all(p) {
        eventually_all_witness(p)
        exists(n: Nat) { forall(m: Nat) { n <= m implies p(m) } }
        let (big_n: Nat) satisfy {
            forall(m: Nat) { big_n <= m implies p(m) }
        }
        density_seq_bounded_above(p)
        is_bounded_above(density_seq(p))
        density_seq_bounded_below(p)
        is_bounded_below(density_seq(p))
        forall(eps: Real) {
            if eps.is_positive {
                pos_gt_zero(eps)
                eps > Real.0
                lt_nat_multiple(from_nat[Real](big_n), eps)
                exists(m: Nat) { from_nat[Real](big_n) < eps * from_nat[Real](m) }
                let (m: Nat) satisfy {
                    from_nat[Real](big_n) < eps * from_nat[Real](m)
                }
                forall(k: Nat) {
                    if m <= k {
                        k <= k.suc
                        nat_lte_trans(m, k, k.suc)
                        m <= k.suc
                        from_nat_real_lte(m, k.suc)
                        from_nat[Real](m) <= from_nat[Real](k.suc)
                        not eps.is_negative
                        lte_mul_nonneg_left(from_nat[Real](m), from_nat[Real](k.suc), eps)
                        eps * from_nat[Real](m) <= eps * from_nat[Real](k.suc)
                        lt_lte_trans(from_nat[Real](big_n), eps * from_nat[Real](m),
                            eps * from_nat[Real](k.suc))
                        from_nat[Real](big_n) < eps * from_nat[Real](k.suc)
                        (range_size(k) = from_nat[Real](k.suc))
                        from_nat[Real](big_n) <= eps * range_size(k)
                        density_seq_ge_of_holds_from(p, big_n, eps, k)
                        Real.1 - eps <= density_seq(p, k)
                        (density_seq(p)(k) = density_seq(p, k))
                        Real.1 - eps <= density_seq(p)(k)
                    }
                    (m <= k implies Real.1 - eps <= density_seq(p)(k))
                }
                tail_inf_is_greatest(density_seq(p), m, Real.1 - eps)
                Real.1 - eps <= tail_inf(density_seq(p), m)
                liminf_ge_tail_inf(density_seq(p), m)
                tail_inf(density_seq(p), m) <= liminf(density_seq(p))
                lte_trans(Real.1 - eps, tail_inf(density_seq(p), m),
                    liminf(density_seq(p)))
                Real.1 - eps <= liminf(density_seq(p))
                lte_add_right(Real.1 - eps, liminf(density_seq(p)), eps)
                (Real.1 - eps) + eps <= liminf(density_seq(p)) + eps
                ((Real.1 - eps) + eps = Real.1)
                (Real.1 <= liminf(density_seq(p)) + eps)
            }
            (eps.is_positive implies Real.1 <= liminf(density_seq(p)) + eps)
        }
        lte_of_lte_add_eps(Real.1, liminf(density_seq(p)))
        Real.1 <= liminf(density_seq(p))
        (lower_density(p) = liminf(density_seq(p)))
        Real.1 <= lower_density(p)
        lower_density_le_upper_density(p)
        lower_density(p) <= upper_density(p)
        upper_density_le_one(p)
        upper_density(p) <= Real.1
        lte_trans(lower_density(p), upper_density(p), Real.1)
        lower_density(p) <= Real.1
        lte_antisymm(lower_density(p), Real.1)
        lower_density(p) = Real.1
    }
}

/// The upper density of a predicate holding from some point on is one.
theorem upper_density_of_eventually_all(p: Nat -> Bool) {
    eventually_all(p) implies upper_density(p) = Real.1
} by {
    if eventually_all(p) {
        lower_density_of_eventually_all(p)
        lower_density(p) = Real.1
        lower_density_le_upper_density(p)
        lower_density(p) <= upper_density(p)
        Real.1 <= upper_density(p)
        upper_density_le_one(p)
        upper_density(p) <= Real.1
        lte_antisymm(upper_density(p), Real.1)
        upper_density(p) = Real.1
    }
}

/// A predicate holding everywhere has density one.
theorem densities_of_all(p: Nat -> Bool) {
    (forall(m: Nat) { p(m) })
        implies lower_density(p) = Real.1 and upper_density(p) = Real.1
} by {
    if forall(m: Nat) { p(m) } {
        eventually_all_of_all(p)
        eventually_all(p)
        lower_density_of_eventually_all(p)
        lower_density(p) = Real.1
        upper_density_of_eventually_all(p)
        upper_density(p) = Real.1
        (lower_density(p) = Real.1 and upper_density(p) = Real.1)
    }
}

/// Two predicates of density one hold together on a set of density one.
///
/// Eventually-true predicates are eventually true together, so the conjunction inherits the
/// density. This is what makes density one behave like a notion of almost-all.
theorem densities_of_and(p: Nat -> Bool, r: Nat -> Bool) {
    eventually_all(p) and eventually_all(r)
        implies lower_density(and_pred(p, r)) = Real.1
            and upper_density(and_pred(p, r)) = Real.1
} by {
    if eventually_all(p) and eventually_all(r) {
        eventually_all_and(p, r)
        eventually_all(and_pred(p, r))
        lower_density_of_eventually_all(and_pred(p, r))
        lower_density(and_pred(p, r)) = Real.1
        upper_density_of_eventually_all(and_pred(p, r))
        upper_density(and_pred(p, r)) = Real.1
        (lower_density(and_pred(p, r)) = Real.1
            and upper_density(and_pred(p, r)) = Real.1)
    }
}
