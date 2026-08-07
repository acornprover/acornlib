from nat import Nat, add_mod, mod_lt, mod_of_decomp, small_mod

numerals Nat

/// Adding the modulus leaves the remainder unchanged.
///
/// The periodicity of the remainder, which nothing in `src/nat/` states. It is what lets a
/// wrap-around argument shift a vertex by a whole turn without changing which class it lands in.
theorem add_mod_self(x: Nat, n: Nat) {
    n != Nat.0 implies (x + n).mod(n) = x.mod(n)
} by {
    if n != Nat.0 {
        add_mod(x, n)
        exists(q: Nat) { q * n + x.mod(n) = x }
        let (q: Nat) satisfy {
            q * n + x.mod(n) = x
        }
        mod_lt(x, n)
        x.mod(n) < n
        (x + n = (q * n + n) + x.mod(n))
        (q.suc * n = q * n + n)
        (x + n = q.suc * n + x.mod(n))
        mod_of_decomp(q.suc, x.mod(n), n)
        (q.suc * n + x.mod(n)).mod(n) = x.mod(n)
        (x + n).mod(n) = x.mod(n)
    }
}

/// A natural below the modulus is unchanged by adding the modulus and reducing.
theorem small_add_mod_self(x: Nat, n: Nat) {
    x < n implies (x + n).mod(n) = x
} by {
    if x < n {
        if n = Nat.0 {
            x < Nat.0
            not (x < Nat.0)
            false
        }
        n != Nat.0
        add_mod_self(x, n)
        (x + n).mod(n) = x.mod(n)
        small_mod(x, n)
        x.mod(n) = x
        (x + n).mod(n) = x
    }
}

/// A decomposition of a successor below the modulus, reduced.
///
/// If `v + 1` reduces to `w` and `v` is below the modulus, then either `w` is `v + 1` outright,
/// or `v + 1` is the modulus itself and `w` is zero. These are the two ways the successor of a
/// vertex of a cycle can behave, and the second is the wrap.
theorem suc_mod_cases(v: Nat, n: Nat, w: Nat) {
    v < n and v.suc.mod(n) = w implies
        (w = v.suc or (v.suc = n and w = Nat.0))
} by {
    if v < n and v.suc.mod(n) = w {
        v.suc <= n
        if v.suc < n {
            small_mod(v.suc, n)
            v.suc.mod(n) = v.suc
            w = v.suc
        }
        if not (v.suc < n) {
            n <= v.suc
            v.suc = n
            (v.suc.mod(n) = n.mod(n))
            (n = Nat.1 * n + Nat.0)
            if n = Nat.0 {
                v < Nat.0
                not (v < Nat.0)
                false
            }
            n != Nat.0
            Nat.0 < n
            mod_of_decomp(Nat.1, Nat.0, n)
            (Nat.1 * n + Nat.0).mod(n) = Nat.0
            n.mod(n) = Nat.0
            w = Nat.0
            (v.suc = n and w = Nat.0)
        }
        (w = v.suc or (v.suc = n and w = Nat.0))
    }
}
