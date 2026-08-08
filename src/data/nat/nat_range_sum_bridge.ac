from nat import Nat
from list import List, sum, map, partial, partial_zero, partial_split_last
from algebra.add_comm_monoid import AddCommMonoid
from data.nat.nat_range_set import range_set
from finite_set import FiniteSet, fs_from_list, finite_set_sum,
    finite_set_sum_eq_list_sum
from data.nat.nat_range_sum import range_sum, range_sum_zero, range_sum_suc

numerals Nat

/// The range sum agrees with the partial sum of the same summand.
///
/// `range_sum` is defined by the recurrence and `partial` by summing over `n.range`; the two
/// have the same base case and the same step, so they agree everywhere. This is the bridge
/// between the range-indexed sums and everything in `src/list/` stated for `partial`.
theorem range_sum_eq_partial[A: AddCommMonoid](f: Nat -> A, n: Nat) {
    range_sum(f, n) = partial(f, n)
} by {
    define p(x: Nat) -> Bool {
        range_sum(f, x) = partial(f, x)
    }
    range_sum_zero(f)
    partial_zero(f)
    range_sum(f, Nat.0) = partial(f, Nat.0)
    p(Nat.0)
    forall(k: Nat) {
        if p(k) {
            range_sum(f, k) = partial(f, k)
            range_sum_suc(f, k)
            range_sum(f, k.suc) = range_sum(f, k) + f(k)
            partial_split_last(f, k)
            partial(f, k.suc) = partial(f, k) + f(k)
            range_sum(f, k.suc) = partial(f, k.suc)
            p(k.suc)
        }
        (p(k) implies p(k.suc))
    }
    p(Nat.0) and forall(k: Nat) {
        p(k) implies p(k.suc)
    }
    Nat.induction(p)
    p(n)
    range_sum(f, n) = partial(f, n)
}

/// The range sum is the sum over the list of indices below the limit.
theorem range_sum_eq_list_sum[A: AddCommMonoid](f: Nat -> A, n: Nat) {
    range_sum(f, n) = sum(map(n.range, f))
} by {
    range_sum_eq_partial(f, n)
    range_sum(f, n) = partial(f, n)
    partial(f, n) = sum(map(n.range, f))
    range_sum(f, n) = sum(map(n.range, f))
}

/// The range sum agrees with the finite-set sum over the indices below the limit.
///
/// The two indexing schemes finally meet: `src/finite_set/sum.ac` sums over a finite set and
/// `src/nat_range_sum.ac` over an initial range, and for the range both give the same total.
/// The index list is duplicate-free, which is what lets the finite-set sum be read off it.
theorem range_sum_eq_finite_set_sum[A: AddCommMonoid](f: Nat -> A, n: Nat) {
    range_sum(f, n) = finite_set_sum(range_set(n), f)
} by {
    n.range.is_unique
    finite_set_sum_eq_list_sum[Nat, A](n.range, f)
    finite_set_sum(fs_from_list(n.range), f) = sum(map(n.range, f))
    range_set(n) = fs_from_list(n.range)
    finite_set_sum(range_set(n), f) = sum(map(n.range, f))
    range_sum_eq_list_sum(f, n)
    range_sum(f, n) = sum(map(n.range, f))
    range_sum(f, n) = finite_set_sum(range_set(n), f)
}
