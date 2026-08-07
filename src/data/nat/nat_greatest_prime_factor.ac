from nat import Nat, divides_lte, has_prime_divisor, divides_trans, lt_or_lte,
    lte_antisymm, lte_and_lt
from number_theory import prime_divides_mul
from data.nat.nat_bounded_max import is_max, has_max, is_max_apply, is_max_is_upper_bound,
    is_upper_bound_of, is_upper_bound_of_intro, max_unique
from data.nat.nat_prime_interval import all_prime_factors_in, all_prime_factors_in_apply,
    all_prime_factors_in_intro

numerals Nat

/// True if `p` is a prime dividing `n`.
///
/// Packaged as a predicate on `Nat` so the maximum construction applies to it.
define is_prime_factor(n: Nat, p: Nat) -> Bool {
    p.is_prime and p.divides(n)
}

/// A prime factor is prime and divides.
theorem is_prime_factor_apply(n: Nat, p: Nat) {
    is_prime_factor(n)(p) implies p.is_prime and p.divides(n)
} by {
    if is_prime_factor(n)(p) {
        is_prime_factor(n, p) = (p.is_prime and p.divides(n))
        p.is_prime and p.divides(n)
    }
}

/// A prime divisor is a prime factor.
theorem is_prime_factor_intro(n: Nat, p: Nat) {
    p.is_prime and p.divides(n) implies is_prime_factor(n)(p)
} by {
    if p.is_prime and p.divides(n) {
        is_prime_factor(n, p) = (p.is_prime and p.divides(n))
        is_prime_factor(n)(p)
    }
}

/// The number itself bounds its prime factors.
///
/// A divisor of a nonzero natural is at most that natural.
theorem prime_factors_bounded(n: Nat) {
    n != Nat.0 implies is_upper_bound_of(is_prime_factor(n), n)
} by {
    if n != Nat.0 {
        forall(p: Nat) {
            if is_prime_factor(n)(p) {
                is_prime_factor_apply(n, p)
                p.divides(n)
                divides_lte(p, n)
                (n = Nat.0 or p <= n)
                p <= n
            }
            (is_prime_factor(n)(p) implies p <= n)
        }
        is_upper_bound_of_intro(is_prime_factor(n), n)
        is_upper_bound_of(is_prime_factor(n), n)
    }
}

/// True if `p` is the largest prime dividing `n`.
///
/// Stated as a predicate rather than a function, because a natural below two has no prime
/// factor at all and there would be nothing for a total function to return.
define is_greatest_prime_factor(n: Nat, p: Nat) -> Bool {
    is_max(is_prime_factor(n), p)
}

/// The greatest prime factor is a prime factor.
theorem is_greatest_prime_factor_apply(n: Nat, p: Nat) {
    is_greatest_prime_factor(n, p) implies p.is_prime and p.divides(n)
} by {
    if is_greatest_prime_factor(n, p) {
        is_greatest_prime_factor(n, p) = is_max(is_prime_factor(n), p)
        is_max(is_prime_factor(n), p)
        is_max_apply(is_prime_factor(n), p)
        is_prime_factor(n)(p)
        is_prime_factor_apply(n, p)
        p.is_prime and p.divides(n)
    }
}

/// No prime factor exceeds the greatest one.
theorem is_greatest_prime_factor_is_greatest(n: Nat, p: Nat, q: Nat) {
    is_greatest_prime_factor(n, p) and q.is_prime and q.divides(n) implies q <= p
} by {
    if is_greatest_prime_factor(n, p) and q.is_prime and q.divides(n) {
        is_greatest_prime_factor(n, p) = is_max(is_prime_factor(n), p)
        is_max(is_prime_factor(n), p)
        is_prime_factor_intro(n, q)
        is_prime_factor(n)(q)
        is_max_is_upper_bound(is_prime_factor(n), p, q)
        q <= p
    }
}

/// Every natural above one has a greatest prime factor.
///
/// It has a prime divisor at all, and its prime divisors are bounded by itself, which is
/// exactly what the maximum construction asks for.
theorem greatest_prime_factor_exists(n: Nat) {
    Nat.1 < n implies exists(p: Nat) { is_greatest_prime_factor(n, p) }
} by {
    if Nat.1 < n {
        has_prime_divisor(n)
        let (q: Nat) satisfy {
            q.is_prime and q.divides(n)
        }
        is_prime_factor_intro(n, q)
        is_prime_factor(n)(q)
        Nat.0 < Nat.1
        lte_and_lt(Nat.0, Nat.1, n)
        Nat.0 < n
        n != Nat.0
        prime_factors_bounded(n)
        is_upper_bound_of(is_prime_factor(n), n)
        has_max(is_prime_factor(n), q, n)
        exists(m: Nat) {
            is_max(is_prime_factor(n), m)
        }
        let (p: Nat) satisfy {
            is_max(is_prime_factor(n), p)
        }
        is_greatest_prime_factor(n, p) = is_max(is_prime_factor(n), p)
        is_greatest_prime_factor(n, p)
        exists(m: Nat) { is_greatest_prime_factor(n, m) }
    }
}

/// The greatest prime factor is unique.
theorem greatest_prime_factor_unique(n: Nat, p: Nat, q: Nat) {
    is_greatest_prime_factor(n, p) and is_greatest_prime_factor(n, q) implies p = q
} by {
    if is_greatest_prime_factor(n, p) and is_greatest_prime_factor(n, q) {
        is_greatest_prime_factor(n, p) = is_max(is_prime_factor(n), p)
        is_greatest_prime_factor(n, q) = is_max(is_prime_factor(n), q)
        max_unique(is_prime_factor(n), p, q)
        p = q
    }
}

/// The greatest prime factor is at most the number.
theorem greatest_prime_factor_le(n: Nat, p: Nat) {
    n != Nat.0 and is_greatest_prime_factor(n, p) implies p <= n
} by {
    if n != Nat.0 and is_greatest_prime_factor(n, p) {
        is_greatest_prime_factor_apply(n, p)
        p.is_prime
        p.divides(n)
        divides_lte(p, n)
        (n = Nat.0 or p <= n)
        p <= n
    }
}

/// A prime factor of a product divides one of the factors.
///
/// Euclid's lemma, restated for the prime factor predicate.
theorem prime_factor_of_product(m: Nat, n: Nat, p: Nat) {
    is_prime_factor(m * n)(p) implies is_prime_factor(m)(p) or is_prime_factor(n)(p)
} by {
    if is_prime_factor(m * n)(p) {
        is_prime_factor_apply(m * n, p)
        p.is_prime
        p.divides(m * n)
        prime_divides_mul(p, m, n)
        (p.divides(m) or p.divides(n))
        if p.divides(m) {
            is_prime_factor_intro(m, p)
            is_prime_factor(m)(p)
        }
        if not p.divides(m) {
            p.divides(n)
            is_prime_factor_intro(n, p)
            is_prime_factor(n)(p)
        }
        (is_prime_factor(m)(p) or is_prime_factor(n)(p))
    }
}

/// The interval condition passes from the factors to the product.
///
/// A prime dividing a product divides one of the factors, so it obeys whichever bound that
/// factor obeys. No coprimality is needed in this direction, or in the other, which is why the
/// statement is about products rather than coprime products.
theorem all_prime_factors_in_mul(m: Nat, n: Nat, lo: Nat, hi: Nat) {
    all_prime_factors_in(m, lo, hi) and all_prime_factors_in(n, lo, hi)
        implies all_prime_factors_in(m * n, lo, hi)
} by {
    if all_prime_factors_in(m, lo, hi) and all_prime_factors_in(n, lo, hi) {
        forall(p: Nat) {
            if p.is_prime and p.divides(m * n) {
                prime_divides_mul(p, m, n)
                (p.divides(m) or p.divides(n))
                if p.divides(m) {
                    all_prime_factors_in_apply(m, lo, hi, p)
                    lo < p and p <= hi
                    lo < p
                }
                if not p.divides(m) {
                    p.divides(n)
                    all_prime_factors_in_apply(n, lo, hi, p)
                    lo < p and p <= hi
                    lo < p
                }
                lo < p
                if p.divides(m) {
                    all_prime_factors_in_apply(m, lo, hi, p)
                    lo < p and p <= hi
                    p <= hi
                }
                if not p.divides(m) {
                    p.divides(n)
                    all_prime_factors_in_apply(n, lo, hi, p)
                    lo < p and p <= hi
                    p <= hi
                }
                p <= hi
                lo < p and p <= hi
            }
            (p.is_prime and p.divides(m * n) implies lo < p and p <= hi)
        }
        all_prime_factors_in_intro(m * n, lo, hi)
        all_prime_factors_in(m * n, lo, hi)
    }
}
