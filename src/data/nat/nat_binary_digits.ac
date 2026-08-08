from nat import Nat, pow_add, pow_one, pow_zero, small_mod, lt_and_lte, lte_and_lt,
    lte_trans, lte_imp_not_lt
from algebra.add_comm_monoid import AddCommMonoid
from data.nat.nat_range_sum import range_sum, range_sum_split, range_sum_congr, shift_fn

numerals Nat

/// Two raised to a power, the place value of the `i`th binary digit.
define two_pow(i: Nat) -> Nat {
    Nat.2.pow(i)
}

/// The place value of the lowest digit is one.
theorem two_pow_zero {
    two_pow(Nat.0) = Nat.1
} by {
    pow_zero(Nat.2)
    Nat.2.pow(Nat.0) = Nat.1
}

/// Each place value is twice the one below it.
theorem two_pow_suc(i: Nat) {
    two_pow(i.suc) = two_pow(i) + two_pow(i)
} by {
    i.suc = i + Nat.1
    pow_add(Nat.2, i, Nat.1)
    Nat.2.pow(i + Nat.1) = Nat.2.pow(i) * Nat.2.pow(Nat.1)
    pow_one(Nat.2)
    Nat.2.pow(Nat.1) = Nat.2
    Nat.2.pow(i + Nat.1) = Nat.2.pow(i) * Nat.2
    Nat.2.pow(i) * Nat.2 = Nat.2.pow(i) + Nat.2.pow(i)
    two_pow(i.suc) = two_pow(i) + two_pow(i)
}

/// Every place value is positive.
theorem two_pow_positive(i: Nat) {
    Nat.0 < two_pow(i)
} by {
    define p(x: Nat) -> Bool {
        Nat.0 < two_pow(x)
    }
    two_pow_zero
    two_pow(Nat.0) = Nat.1
    Nat.0 < Nat.1
    p(Nat.0)
    forall(k: Nat) {
        if p(k) {
            Nat.0 < two_pow(k)
            two_pow_suc(k)
            two_pow(k.suc) = two_pow(k) + two_pow(k)
            two_pow(k) <= two_pow(k) + two_pow(k)
            lt_and_lte(Nat.0, two_pow(k), two_pow(k.suc))
            Nat.0 < two_pow(k.suc)
            p(k.suc)
        }
        (p(k) implies p(k.suc))
    }
    p(Nat.0) and forall(k: Nat) {
        p(k) implies p(k.suc)
    }
    Nat.induction(p)
    p(i)
}

/// Each place value is strictly below the next.
theorem two_pow_lt_suc(i: Nat) {
    two_pow(i) < two_pow(i.suc)
} by {
    two_pow_positive(i)
    Nat.0 < two_pow(i)
    two_pow_suc(i)
    two_pow(i.suc) = two_pow(i) + two_pow(i)
    two_pow(i) < two_pow(i) + two_pow(i)
    two_pow(i) < two_pow(i.suc)
}

/// True if the `i`th binary digit of `n` is one.
///
/// Read by discarding the places above `i` and asking whether what remains reaches the place
/// value. Stated through `mod` and a comparison rather than through division, so it needs
/// nothing beyond the public arithmetic on the naturals.
define bit_set(n: Nat, i: Nat) -> Bool {
    two_pow(i) <= n.mod(two_pow(i.suc))
}

/// The digit condition, with the truncation carried out.
theorem bit_set_eq(n: Nat, i: Nat) {
    bit_set(n, i) = (two_pow(i) <= n.mod(two_pow(i.suc)))
}

/// A number below a place value has a zero digit there.
///
/// Everything strictly below `2^i` is written using digits below position `i`, and truncating
/// above position `i` leaves it unchanged.
theorem bit_clear_of_lt(n: Nat, i: Nat) {
    n < two_pow(i) implies not bit_set(n, i)
} by {
    if n < two_pow(i) {
        two_pow_lt_suc(i)
        two_pow(i) < two_pow(i.suc)
        n < two_pow(i.suc)
        small_mod(n, two_pow(i.suc))
        n.mod(two_pow(i.suc)) = n
        if bit_set(n, i) {
            bit_set_eq(n, i)
            two_pow(i) <= n.mod(two_pow(i.suc))
            two_pow(i) <= n
            lte_imp_not_lt(two_pow(i), n)
            not (n < two_pow(i))
            false
        }
        not bit_set(n, i)
    }
}

/// Adding a place value to something below it sets exactly that digit.
///
/// This is the pairing that matches the two halves of a range of length `2^(i+1)`: the lower
/// half has digit `i` clear, and the upper half is the lower half with `2^i` added.
theorem bit_set_of_add(r: Nat, i: Nat) {
    r < two_pow(i) implies bit_set(two_pow(i) + r, i)
} by {
    if r < two_pow(i) {
        two_pow_suc(i)
        two_pow(i.suc) = two_pow(i) + two_pow(i)
        two_pow(i) + r < two_pow(i) + two_pow(i)
        two_pow(i) + r < two_pow(i.suc)
        small_mod(two_pow(i) + r, two_pow(i.suc))
        (two_pow(i) + r).mod(two_pow(i.suc)) = two_pow(i) + r
        two_pow(i) <= two_pow(i) + r
        two_pow(i) <= (two_pow(i) + r).mod(two_pow(i.suc))
        bit_set_eq(two_pow(i) + r, i)
        bit_set(two_pow(i) + r, i)
    }
}

/// The two halves of a binary range are told apart by the top digit.
///
/// Below `2^i` the digit is clear, and every index of the upper half is `2^i` plus an index
/// of the lower half, where the digit is set. This is what makes a summand described by the
/// binary expansion of its index split cleanly along the halves.
theorem binary_halves_differ(i: Nat, r: Nat) {
    r < two_pow(i) implies not bit_set(r, i) and bit_set(two_pow(i) + r, i)
} by {
    if r < two_pow(i) {
        bit_clear_of_lt(r, i)
        not bit_set(r, i)
        bit_set_of_add(r, i)
        bit_set(two_pow(i) + r, i)
        not bit_set(r, i) and bit_set(two_pow(i) + r, i)
    }
}

/// The lowest digit of a place value below it is clear.
theorem bit_clear_zero(i: Nat) {
    not bit_set(Nat.0, i)
} by {
    two_pow_positive(i)
    Nat.0 < two_pow(i)
    bit_clear_of_lt(Nat.0, i)
    not bit_set(Nat.0, i)
}

/// A range of length `2^(i+1)` splits into two halves of length `2^i`.
///
/// The upper half is the lower half of the summand shifted by `2^i`, which by
/// `bit_set_of_add` is exactly where the `i`th binary digit is set.
theorem range_sum_binary_split[A: AddCommMonoid](f: Nat -> A, i: Nat) {
    range_sum(f, two_pow(i.suc)) =
        range_sum(f, two_pow(i)) + range_sum(shift_fn(f, two_pow(i)), two_pow(i))
} by {
    two_pow_suc(i)
    two_pow(i.suc) = two_pow(i) + two_pow(i)
    range_sum_split(f, two_pow(i), two_pow(i))
    (range_sum(f, two_pow(i) + two_pow(i)) =
        range_sum(f, two_pow(i)) + range_sum(shift_fn(f, two_pow(i)), two_pow(i)))
    (range_sum(f, two_pow(i.suc)) =
        range_sum(f, two_pow(i)) + range_sum(shift_fn(f, two_pow(i)), two_pow(i)))
}

/// A summand that ignores the `i`th digit agrees with the shifted summand on the lower half.
///
/// The bridge from the split above to a summand described by the binary expansion: on the
/// lower half indices have digit `i` clear, and every upper-half index is a lower-half index
/// with that digit set.
theorem range_sum_halves_congr[A: AddCommMonoid](
    f: Nat -> A, g: Nat -> A, i: Nat
) {
    (forall(r: Nat) {
        r < two_pow(i) implies f(two_pow(i) + r) = g(r)
    }) implies range_sum(shift_fn(f, two_pow(i)), two_pow(i)) = range_sum(g, two_pow(i))
} by {
    if forall(r: Nat) { r < two_pow(i) implies f(two_pow(i) + r) = g(r) } {
        forall(r: Nat) {
            if r < two_pow(i) {
                f(two_pow(i) + r) = g(r)
                shift_fn(f, two_pow(i), r) = f(two_pow(i) + r)
                shift_fn(f, two_pow(i), r) = g(r)
            }
            (r < two_pow(i) implies shift_fn(f, two_pow(i), r) = g(r))
        }
        range_sum_congr(shift_fn(f, two_pow(i)), g, two_pow(i))
        range_sum(shift_fn(f, two_pow(i)), two_pow(i)) = range_sum(g, two_pow(i))
    }
}
