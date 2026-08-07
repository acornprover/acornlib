from nat import Nat, is_min, has_min, is_min_apply, is_min_false_below, false_below,
    false_below_apply, lt_or_lte, lte_antisymm, lt_imp_lte_suc, lte_cancel_suc,
    lte_and_lt, lt_and_lte, only_zero_lte_zero

numerals Nat

/// True if `b` bounds every natural satisfying `f`.
define is_upper_bound_of(f: Nat -> Bool, b: Nat) -> Bool {
    forall(n: Nat) {
        f(n) implies n <= b
    }
}

/// Such a bound dominates each satisfying natural.
theorem is_upper_bound_of_apply(f: Nat -> Bool, b: Nat, n: Nat) {
    is_upper_bound_of(f, b) and f(n) implies n <= b
} by {
    if is_upper_bound_of(f, b) and f(n) {
        is_upper_bound_of(f, b) = forall(k: Nat) {
            f(k) implies k <= b
        }
        forall(k: Nat) {
            f(k) implies k <= b
        }
        (f(n) implies n <= b)
        n <= b
    }
}

/// A pointwise bound is an upper bound.
theorem is_upper_bound_of_intro(f: Nat -> Bool, b: Nat) {
    (forall(n: Nat) { f(n) implies n <= b }) implies is_upper_bound_of(f, b)
} by {
    if forall(n: Nat) { f(n) implies n <= b } {
        is_upper_bound_of(f, b) = forall(k: Nat) {
            f(k) implies k <= b
        }
        is_upper_bound_of(f, b)
    }
}

/// True if `m` is the largest natural for which `f` holds.
///
/// The dual of `is_min`. `src/nat/` supplies the least element of a nonempty predicate but
/// nothing about the greatest, which is what a maximisation over a bounded family needs.
define is_max(f: Nat -> Bool, m: Nat) -> Bool {
    f(m) and is_upper_bound_of(f, m)
}

/// The maximum satisfies the predicate.
theorem is_max_apply(f: Nat -> Bool, m: Nat) {
    is_max(f, m) implies f(m)
} by {
    if is_max(f, m) {
        is_max(f, m) = (f(m) and is_upper_bound_of(f, m))
        f(m)
    }
}

/// Nothing satisfying the predicate exceeds the maximum.
theorem is_max_is_upper_bound(f: Nat -> Bool, m: Nat, n: Nat) {
    is_max(f, m) and f(n) implies n <= m
} by {
    if is_max(f, m) and f(n) {
        is_max(f, m) = (f(m) and is_upper_bound_of(f, m))
        is_upper_bound_of(f, m)
        is_upper_bound_of_apply(f, m, n)
        n <= m
    }
}

/// The two conditions give a maximum.
theorem is_max_intro(f: Nat -> Bool, m: Nat) {
    f(m) and is_upper_bound_of(f, m) implies is_max(f, m)
} by {
    if f(m) and is_upper_bound_of(f, m) {
        is_max(f, m) = (f(m) and is_upper_bound_of(f, m))
        is_max(f, m)
    }
}

/// A satisfiable predicate with an upper bound has a greatest satisfying natural.
///
/// Obtained from the least element construction applied to the upper bounds rather than to the
/// predicate itself. The least upper bound must itself satisfy the predicate: were it not to,
/// its predecessor would bound the predicate too, contradicting leastness.
theorem has_max(f: Nat -> Bool, witness: Nat, bound: Nat) {
    f(witness) and is_upper_bound_of(f, bound) implies exists(m: Nat) { is_max(f, m) }
} by {
    if f(witness) and is_upper_bound_of(f, bound) {
        has_min(is_upper_bound_of(f), bound)
        exists(k: Nat) {
            is_min(is_upper_bound_of(f), k)
        }
        let (m: Nat) satisfy {
            is_min(is_upper_bound_of(f), m)
        }
        is_min_apply(is_upper_bound_of(f), m)
        is_upper_bound_of(f)(m)
        is_upper_bound_of(f, m)
        if not f(m) {
            is_upper_bound_of_apply(f, m, witness)
            witness <= m
            if m = Nat.0 {
                only_zero_lte_zero(witness)
                witness = Nat.0
                f(Nat.0)
                f(m)
                false
            }
            m != Nat.0
            let (j: Nat) satisfy {
                j.suc = m
            }
            forall(n: Nat) {
                if f(n) {
                    is_upper_bound_of_apply(f, m, n)
                    n <= m
                    if n = m {
                        f(m)
                        false
                    }
                    n != m
                    n < m
                    lt_imp_lte_suc(n, m)
                    n.suc <= j.suc
                    lte_cancel_suc(n, j)
                    n <= j
                }
                (f(n) implies n <= j)
            }
            is_upper_bound_of_intro(f, j)
            is_upper_bound_of(f, j)
            is_upper_bound_of(f)(j)
            is_min_false_below(is_upper_bound_of(f), m)
            false_below(is_upper_bound_of(f), m)
            j < j.suc
            j < m
            false_below_apply(is_upper_bound_of(f), m, j)
            not is_upper_bound_of(f)(j)
            false
        }
        f(m)
        is_max_intro(f, m)
        is_max(f, m)
        exists(k: Nat) { is_max(f, k) }
    }
}

/// The maximum is unique.
theorem max_unique(f: Nat -> Bool, m: Nat, m2: Nat) {
    is_max(f, m) and is_max(f, m2) implies m = m2
} by {
    if is_max(f, m) and is_max(f, m2) {
        is_max_apply(f, m)
        f(m)
        is_max_apply(f, m2)
        f(m2)
        is_max_is_upper_bound(f, m, m2)
        m2 <= m
        is_max_is_upper_bound(f, m2, m)
        m <= m2
        lte_antisymm(m, m2)
        m = m2
    }
}
