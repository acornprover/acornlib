from nat import Nat
from real import Real, is_upper_bound, is_lower_bound, lte_trans
from data.nat.nat_density import density_seq, density_seq_le_one, density_seq_nonnegative
from analysis import tail_sup, tail_sup_is_least, is_bounded_above,
    is_bounded_above_intro, limsup, limsup_le_tail_sup, is_bounded_below,
    is_bounded_below_intro, liminf, tail_inf, tail_inf_is_greatest, liminf_ge_tail_inf,
    liminf_le_limsup

/// The density sequence never exceeds one.
theorem density_seq_bounded_above(p: Nat -> Bool) {
    is_bounded_above(density_seq(p))
} by {
    forall(n: Nat) {
        density_seq_le_one(p, n)
        density_seq(p, n) <= Real.1
        (density_seq(p)(n) = density_seq(p, n))
        density_seq(p)(n) <= Real.1
    }
    (is_upper_bound(density_seq(p), Real.1) = forall(n: Nat) {
        density_seq(p)(n) <= Real.1
    })
    is_upper_bound(density_seq(p), Real.1)
    is_bounded_above_intro(density_seq(p), Real.1)
    is_bounded_above(density_seq(p))
}

/// The density sequence is never negative.
theorem density_seq_bounded_below(p: Nat -> Bool) {
    is_bounded_below(density_seq(p))
} by {
    forall(n: Nat) {
        density_seq_nonnegative(p, n)
        Real.0 <= density_seq(p, n)
        (density_seq(p)(n) = density_seq(p, n))
        Real.0 <= density_seq(p)(n)
    }
    (is_lower_bound(density_seq(p), Real.0) = forall(n: Nat) {
        Real.0 <= density_seq(p)(n)
    })
    is_lower_bound(density_seq(p), Real.0)
    is_bounded_below_intro(density_seq(p), Real.0)
    is_bounded_below(density_seq(p))
}

/// The upper density of a predicate on the naturals.
///
/// The limit superior of the density sequence. Unlike the natural density this always exists,
/// since the density sequence lies in the unit interval and so is bounded on both sides.
define upper_density(p: Nat -> Bool) -> Real {
    limsup(density_seq(p))
}

/// The lower density of a predicate on the naturals.
define lower_density(p: Nat -> Bool) -> Real {
    liminf(density_seq(p))
}

/// The upper density is at most one.
theorem upper_density_le_one(p: Nat -> Bool) {
    upper_density(p) <= Real.1
} by {
    density_seq_bounded_above(p)
    is_bounded_above(density_seq(p))
    density_seq_bounded_below(p)
    is_bounded_below(density_seq(p))
    limsup_le_tail_sup(density_seq(p), Nat.0)
    limsup(density_seq(p)) <= tail_sup(density_seq(p), Nat.0)
    forall(k: Nat) {
        density_seq_le_one(p, k)
        density_seq(p, k) <= Real.1
        (density_seq(p)(k) = density_seq(p, k))
        density_seq(p)(k) <= Real.1
        (Nat.0 <= k implies density_seq(p)(k) <= Real.1)
    }
    tail_sup_is_least(density_seq(p), Nat.0, Real.1)
    tail_sup(density_seq(p), Nat.0) <= Real.1
    lte_trans(limsup(density_seq(p)), tail_sup(density_seq(p), Nat.0), Real.1)
    limsup(density_seq(p)) <= Real.1
    (upper_density(p) = limsup(density_seq(p)))
    upper_density(p) <= Real.1
}

/// The lower density is at least zero.
theorem lower_density_nonneg(p: Nat -> Bool) {
    Real.0 <= lower_density(p)
} by {
    density_seq_bounded_above(p)
    is_bounded_above(density_seq(p))
    density_seq_bounded_below(p)
    is_bounded_below(density_seq(p))
    forall(k: Nat) {
        density_seq_nonnegative(p, k)
        Real.0 <= density_seq(p, k)
        (density_seq(p)(k) = density_seq(p, k))
        Real.0 <= density_seq(p)(k)
        (Nat.0 <= k implies Real.0 <= density_seq(p)(k))
    }
    tail_inf_is_greatest(density_seq(p), Nat.0, Real.0)
    Real.0 <= tail_inf(density_seq(p), Nat.0)
    liminf_ge_tail_inf(density_seq(p), Nat.0)
    tail_inf(density_seq(p), Nat.0) <= liminf(density_seq(p))
    lte_trans(Real.0, tail_inf(density_seq(p), Nat.0), liminf(density_seq(p)))
    Real.0 <= liminf(density_seq(p))
    (lower_density(p) = liminf(density_seq(p)))
    Real.0 <= lower_density(p)
}

/// The lower density never exceeds the upper density.
///
/// Immediate from the comparison of the limit inferior and the limit superior, which applies
/// because the density sequence is bounded on both sides.
theorem lower_density_le_upper_density(p: Nat -> Bool) {
    lower_density(p) <= upper_density(p)
} by {
    density_seq_bounded_above(p)
    is_bounded_above(density_seq(p))
    density_seq_bounded_below(p)
    is_bounded_below(density_seq(p))
    liminf_le_limsup(density_seq(p))
    liminf(density_seq(p)) <= limsup(density_seq(p))
    (lower_density(p) = liminf(density_seq(p)))
    (upper_density(p) = limsup(density_seq(p)))
    lower_density(p) <= upper_density(p)
}

/// Both densities lie in the unit interval.
theorem densities_in_unit_interval(p: Nat -> Bool) {
    Real.0 <= lower_density(p) and lower_density(p) <= upper_density(p)
        and upper_density(p) <= Real.1
} by {
    lower_density_nonneg(p)
    Real.0 <= lower_density(p)
    lower_density_le_upper_density(p)
    lower_density(p) <= upper_density(p)
    upper_density_le_one(p)
    upper_density(p) <= Real.1
    (Real.0 <= lower_density(p) and lower_density(p) <= upper_density(p)
        and upper_density(p) <= Real.1)
}
