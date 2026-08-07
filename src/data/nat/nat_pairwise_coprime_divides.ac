from nat import Nat, divides_self, lt_and_lte
from list import List, product
from number_theory import coprime_with_all, pairwise_coprime, pairwise_coprime_cons_imp,
    coprime_with_all_imp_coprime_product, coprime_mul, coprime_comm, coprime_one_right
from data.nat.nat_range_prod import range_prod, range_prod_zero, range_prod_suc
from data.nat.nat_least_non_divisor import coprime_divisors_product_divides

numerals Nat

/// True if every element of a list divides the given natural.
///
/// Named rather than written as a quantifier at each use, so that the statement below is one
/// level deep and the induction reads off the cons case directly.
define all_divide(list: List[Nat], n: Nat) -> Bool {
    forall(d: Nat) {
        list.contains(d) implies d.divides(n)
    }
}

/// A cons list of divisors gives a head divisor and a tail of divisors.
///
/// Stated through membership rather than as a recursion returning `Bool`. A recursion whose cons
/// case is a conjunction does not unfold as an equation, which is what the residue-class work
/// ran into as well.
theorem all_divide_cons_imp(head: Nat, tail: List[Nat], n: Nat) {
    all_divide(List.cons(head, tail), n) implies head.divides(n) and all_divide(tail, n)
} by {
    if all_divide(List.cons(head, tail), n) {
        (all_divide(List.cons(head, tail), n) = forall(d: Nat) {
            List.cons(head, tail).contains(d) implies d.divides(n)
        })
        List.cons(head, tail).contains(head)
        (List.cons(head, tail).contains(head) implies head.divides(n))
        head.divides(n)
        forall(d: Nat) {
            if tail.contains(d) {
                List.cons(head, tail).contains(d)
                (List.cons(head, tail).contains(d) implies d.divides(n))
                d.divides(n)
            }
            (tail.contains(d) implies d.divides(n))
        }
        (all_divide(tail, n) = forall(d: Nat) {
            tail.contains(d) implies d.divides(n)
        })
        all_divide(tail, n)
        (head.divides(n) and all_divide(tail, n))
    }
}

/// The product of a pairwise coprime list of divisors is a divisor.
///
/// The step a divisibility argument over several primes turns on: knowing each of them divides a
/// number gives no bound on their product unless they are coprime, and then the product divides
/// too. Induction along the list, with the head coprime to the product of the tail because it is
/// coprime to each of its factors.
theorem pairwise_coprime_product_divides(list: List[Nat], n: Nat) {
    pairwise_coprime(list) and all_divide(list, n) implies product[Nat](list).divides(n)
} by {
    define p(l: List[Nat]) -> Bool {
        pairwise_coprime(l) and all_divide(l, n) implies product[Nat](l).divides(n)
    }
    (product[Nat](List.nil[Nat]) = Nat.1)
    Nat.1.divides(n)
    p(List.nil[Nat])
    forall(head: Nat, tail: List[Nat]) {
        if p(tail) {
            if pairwise_coprime(List.cons(head, tail))
                and all_divide(List.cons(head, tail), n) {
                pairwise_coprime_cons_imp(head, tail)
                (coprime_with_all(head, tail) and pairwise_coprime(tail))
                all_divide_cons_imp(head, tail, n)
                (head.divides(n) and all_divide(tail, n))
                (pairwise_coprime(tail) and all_divide(tail, n)
                    implies product[Nat](tail).divides(n))
                product[Nat](tail).divides(n)
                coprime_with_all_imp_coprime_product(head, tail)
                head.coprime(product[Nat](tail))
                coprime_divisors_product_divides(head, product[Nat](tail), n)
                (head * product[Nat](tail)).divides(n)
                (product[Nat](List.cons(head, tail)) = head * product[Nat](tail))
                product[Nat](List.cons(head, tail)).divides(n)
            }
            p(List.cons(head, tail))
        }
        (p(tail) implies p(List.cons(head, tail)))
    }
    p(List.nil[Nat]) and forall(h: Nat, t: List[Nat]) {
        p(t) implies p(List.cons(h, t))
    }
    List.induction(p)
    p(list)
}

/// True if the factors below the limit are pairwise coprime.
define range_pairwise_coprime(f: Nat -> Nat, n: Nat) -> Bool {
    forall(i: Nat, j: Nat) {
        i < n and j < n and i != j implies f(i).coprime(f(j))
    }
}

/// A natural coprime to every factor is coprime to the range product.
theorem coprime_range_prod(f: Nat -> Nat, a: Nat, n: Nat) {
    (forall(i: Nat) { i < n implies a.coprime(f(i)) })
        implies a.coprime(range_prod(f, n))
} by {
    if forall(i: Nat) { i < n implies a.coprime(f(i)) } {
        define p(x: Nat) -> Bool {
            x <= n implies a.coprime(range_prod(f, x))
        }
        range_prod_zero(f)
        (range_prod(f, Nat.0) = Nat.1)
        coprime_one_right(a)
        a.coprime(Nat.1)
        p(Nat.0)
        forall(k: Nat) {
            if p(k) {
                if k.suc <= n {
                    k < k.suc
                    lt_and_lte(k, k.suc, n)
                    k < n
                    a.coprime(f(k))
                    k <= n
                    a.coprime(range_prod(f, k))
                    coprime_mul(a, range_prod(f, k), f(k))
                    a.coprime(range_prod(f, k) * f(k))
                    range_prod_suc(f, k)
                    (range_prod(f, k.suc) = range_prod(f, k) * f(k))
                    a.coprime(range_prod(f, k.suc))
                }
                p(k.suc)
            }
            (p(k) implies p(k.suc))
        }
        p(Nat.0) and forall(k: Nat) {
            p(k) implies p(k.suc)
        }
        Nat.induction(p)
        p(n)
        n <= n
        a.coprime(range_prod(f, n))
    }
}

/// The product over a range of pairwise coprime divisors is a divisor.
///
/// The range form, which is what an argument over an interval uses. Each new factor is coprime to
/// the product of the earlier ones because it is coprime to each of them.
theorem range_pairwise_coprime_product_divides(f: Nat -> Nat, n: Nat, x: Nat) {
    range_pairwise_coprime(f, n) and (forall(i: Nat) { i < n implies f(i).divides(x) })
        implies range_prod(f, n).divides(x)
} by {
    if range_pairwise_coprime(f, n)
        and forall(i: Nat) { i < n implies f(i).divides(x) } {
        (range_pairwise_coprime(f, n) = forall(i: Nat, j: Nat) {
            i < n and j < n and i != j implies f(i).coprime(f(j))
        })
        define p(y: Nat) -> Bool {
            y <= n implies range_prod(f, y).divides(x)
        }
        range_prod_zero(f)
        (range_prod(f, Nat.0) = Nat.1)
        Nat.1.divides(x)
        p(Nat.0)
        forall(k: Nat) {
            if p(k) {
                if k.suc <= n {
                    k < k.suc
                    lt_and_lte(k, k.suc, n)
                    k < n
                    f(k).divides(x)
                    k <= n
                    range_prod(f, k).divides(x)
                    forall(i: Nat) {
                        if i < k {
                            i < k.suc
                            lt_and_lte(i, k.suc, n)
                            i < n
                            i != k
                            (i < n and k < n and i != k implies f(i).coprime(f(k)))
                            f(i).coprime(f(k))
                            coprime_comm(f(i), f(k))
                            f(k).coprime(f(i))
                        }
                        (i < k implies f(k).coprime(f(i)))
                    }
                    coprime_range_prod(f, f(k), k)
                    f(k).coprime(range_prod(f, k))
                    coprime_divisors_product_divides(f(k), range_prod(f, k), x)
                    (f(k) * range_prod(f, k)).divides(x)
                    range_prod_suc(f, k)
                    (range_prod(f, k.suc) = range_prod(f, k) * f(k))
                    (f(k) * range_prod(f, k) = range_prod(f, k) * f(k))
                    range_prod(f, k.suc).divides(x)
                }
                p(k.suc)
            }
            (p(k) implies p(k.suc))
        }
        p(Nat.0) and forall(k: Nat) {
            p(k) implies p(k.suc)
        }
        Nat.induction(p)
        p(n)
        n <= n
        range_prod(f, n).divides(x)
    }
}
