from nat import Nat, mul_assoc, mul_comm
from data.nat.nat_mul_rearrange import mul_self_mul_self

numerals Nat

/// True if a natural is the square of some natural.
define is_square(n: Nat) -> Bool {
    exists(r: Nat) {
        n = r * r
    }
}

/// A root witnesses squareness.
theorem is_square_intro(n: Nat, r: Nat) {
    n = r * r implies is_square(n)
} by {
    if n = r * r {
        is_square(n) = exists(s: Nat) {
            n = s * s
        }
        exists(s: Nat) {
            n = s * s
        }
        is_square(n)
    }
}

/// A square has a root.
theorem is_square_witness(n: Nat) {
    is_square(n) implies exists(r: Nat) { n = r * r }
} by {
    if is_square(n) {
        is_square(n) = exists(s: Nat) {
            n = s * s
        }
        exists(s: Nat) {
            n = s * s
        }
    }
}

/// Zero is a square.
theorem zero_is_square {
    is_square(Nat.0)
} by {
    Nat.0 = Nat.0 * Nat.0
    is_square_intro(Nat.0, Nat.0)
}

/// One is a square.
theorem one_is_square {
    is_square(Nat.1)
} by {
    Nat.1 = Nat.1 * Nat.1
    is_square_intro(Nat.1, Nat.1)
}

/// The square of any natural is a square.
theorem mul_self_is_square(r: Nat) {
    is_square(r * r)
} by {
    r * r = r * r
    is_square_intro(r * r, r)
}

/// A product of squares is a square.
///
/// No coprimality is needed in this direction: the roots simply multiply.
theorem square_mul_square(m: Nat, n: Nat) {
    is_square(m) and is_square(n) implies is_square(m * n)
} by {
    if is_square(m) and is_square(n) {
        is_square_witness(m)
        let (a: Nat) satisfy {
            m = a * a
        }
        is_square_witness(n)
        let (b: Nat) satisfy {
            n = b * b
        }
        m * n = (a * a) * (b * b)
        mul_self_mul_self(a, b)
        (a * a) * (b * b) = (a * b) * (a * b)
        m * n = (a * b) * (a * b)
        is_square_intro(m * n, a * b)
        is_square(m * n)
    }
}

/// A square times itself is a fourth power, and in particular a square.
theorem square_of_square(n: Nat) {
    is_square(n) implies is_square(n * n)
} by {
    if is_square(n) {
        mul_self_is_square(n)
    }
}

/// Multiplying a square by one leaves it a square.
theorem square_mul_one(n: Nat) {
    is_square(n) implies is_square(n * Nat.1)
} by {
    if is_square(n) {
        n * Nat.1 = n
        is_square(n * Nat.1)
    }
}
