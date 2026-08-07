from nat import Nat, lt_and_lte, lte_and_lt
from list import List, range_contains_of_lt, lt_of_range_contains, length_range
from finite_set import FiniteSet, fs_from_list, finite_set_ext,
    finite_set_from_unique_list_cardinality_is_length
from data.finite.finite_set_card import fs_card, fs_card_eq_of_cardinality_is
from data.finite.finite_set_membership import fs_from_list_contains_eq
from data.finite.finite_set_subset_intro import fs_subset_eq_intro

numerals Nat

/// The finite set of naturals below a limit.
///
/// This is the vertex set of a finite path or cycle, the index set of a range sum, and the
/// domain of a counting function, so it is stated once here rather than in any one of them.
define range_set(n: Nat) -> FiniteSet[Nat] {
    fs_from_list(n.range)
}

/// A natural lies in the range set exactly when it is below the limit.
theorem range_set_contains_eq(n: Nat, x: Nat) {
    range_set(n).contains(x) = (x < n)
} by {
    fs_from_list_contains_eq(n.range, x)
    range_set(n).contains(x) = n.range.contains(x)
    if n.range.contains(x) {
        lt_of_range_contains(n, x)
        x < n
    }
    if x < n {
        range_contains_of_lt(n, x)
        n.range.contains(x)
    }
    (n.range.contains(x) implies x < n)
    (x < n implies n.range.contains(x))
    n.range.contains(x) = (x < n)
}

/// Everything below the limit is a member.
theorem range_set_contains(n: Nat, x: Nat) {
    x < n implies range_set(n).contains(x)
} by {
    if x < n {
        range_set_contains_eq(n, x)
        range_set(n).contains(x)
    }
}

/// A member is below the limit.
theorem range_set_lt(n: Nat, x: Nat) {
    range_set(n).contains(x) implies x < n
} by {
    if range_set(n).contains(x) {
        range_set_contains_eq(n, x)
        x < n
    }
}

/// Range sets are nested by their limits.
theorem range_set_subset(m: Nat, n: Nat) {
    m <= n implies range_set(m).subset_eq(range_set(n))
} by {
    if m <= n {
        forall(x: Nat) {
            if range_set(m).contains(x) {
                range_set_lt(m, x)
                x < m
                lt_and_lte(x, m, n)
                x < n
                range_set_contains(n, x)
                range_set(n).contains(x)
            }
            (range_set(m).contains(x) implies range_set(n).contains(x))
        }
        fs_subset_eq_intro(range_set(m), range_set(n))
        range_set(m).subset_eq(range_set(n))
    }
}

/// The range set of a successor is the previous one together with the new limit.
///
/// Both values of the right-hand side are treated separately: proof search does not try both
/// truth values of a boolean on its own, and the false case is the one that needs the step
/// from `n <= x` to `n.suc <= x`.
theorem range_set_suc(n: Nat, x: Nat) {
    range_set(n.suc).contains(x) = (range_set(n).contains(x) or x = n)
} by {
    if (range_set(n).contains(x) or x = n) {
        if range_set(n).contains(x) {
            range_set_lt(n, x)
            x < n
            n < n.suc
            lt_and_lte(x, n, n.suc)
            x < n.suc
        }
        if x = n {
            n < n.suc
            x < n.suc
        }
        x < n.suc
        range_set_contains(n.suc, x)
        range_set(n.suc).contains(x)
        range_set(n.suc).contains(x) = (range_set(n).contains(x) or x = n)
    }
    if not (range_set(n).contains(x) or x = n) {
        not range_set(n).contains(x)
        x != n
        if x < n {
            range_set_contains(n, x)
            range_set(n).contains(x)
            false
        }
        not (x < n)
        n <= x
        (n < x or n = x)
        n < x
        n.suc <= x
        not (x < n.suc)
        if range_set(n.suc).contains(x) {
            range_set_lt(n.suc, x)
            x < n.suc
            false
        }
        not range_set(n.suc).contains(x)
        range_set(n.suc).contains(x) = (range_set(n).contains(x) or x = n)
    }
}

/// The range set below `n` has exactly `n` elements.
theorem range_set_card(n: Nat) {
    fs_card(range_set(n)) = n
} by {
    n.range.is_unique
    finite_set_from_unique_list_cardinality_is_length(n.range)
    fs_from_list(n.range).cardinality_is(n.range.length)
    length_range(n)
    n.range.length = n
    range_set(n).cardinality_is(n)
    fs_card_eq_of_cardinality_is(range_set(n), n)
    fs_card(range_set(n)) = n
}
