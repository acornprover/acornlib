from nat import Nat, from_nat
from list import List
from pair import Pair
from real import Real, lte_trans, mul_left_cancel, lte_add_left
from data.nat.nat_counting_function import count_upto, none_pred, count_upto_none
from data.nat.nat_density import range_size, range_size_positive, density_seq
from analysis import tail_sup, tail_sup_is_least, is_bounded_above, limsup,
    limsup_le_tail_sup, is_bounded_below
from data.nat.nat_density_extremes import upper_density, density_seq_bounded_above,
    density_seq_bounded_below
from data.nat.nat_residue_count import in_residue_class
from data.nat.nat_residue_density_value import residue_class_density, from_nat_modulus_pos
from data.nat.nat_density_subadditive import or_pred, upper_density_subadditive

numerals Nat

/// The predicate that holds nowhere has density fraction zero.
theorem density_seq_none(n: Nat) {
    density_seq(none_pred, n) = Real.0
} by {
    count_upto_none(n.suc)
    count_upto(none_pred, n.suc) = Nat.0
    (from_nat[Real](Nat.0) = Real.0)
    (density_seq(none_pred, n)
        = from_nat[Real](count_upto(none_pred, n.suc)) / range_size(n))
    (density_seq(none_pred, n) = Real.0 / range_size(n))
    range_size_positive(n)
    Real.0 < range_size(n)
    range_size(n) != Real.0
    (Real.0 / range_size(n) = Real.0)
    density_seq(none_pred, n) = Real.0
}

/// The predicate that holds nowhere has upper density zero.
theorem upper_density_none {
    upper_density(none_pred) <= Real.0
} by {
    density_seq_bounded_above(none_pred)
    is_bounded_above(density_seq(none_pred))
    density_seq_bounded_below(none_pred)
    is_bounded_below(density_seq(none_pred))
    forall(k: Nat) {
        density_seq_none(k)
        density_seq(none_pred, k) = Real.0
        (density_seq(none_pred)(k) = density_seq(none_pred, k))
        density_seq(none_pred)(k) <= Real.0
        (Nat.0 <= k implies density_seq(none_pred)(k) <= Real.0)
    }
    tail_sup_is_least(density_seq(none_pred), Nat.0, Real.0)
    tail_sup(density_seq(none_pred), Nat.0) <= Real.0
    limsup_le_tail_sup(density_seq(none_pred), Nat.0)
    limsup(density_seq(none_pred)) <= tail_sup(density_seq(none_pred), Nat.0)
    lte_trans(limsup(density_seq(none_pred)),
        tail_sup(density_seq(none_pred), Nat.0), Real.0)
    limsup(density_seq(none_pred)) <= Real.0
    (upper_density(none_pred) = limsup(density_seq(none_pred)))
    upper_density(none_pred) <= Real.0
}

/// The upper density of a residue class in division form.
///
/// `residue_class_density` states this cross-multiplied, which is the form that avoids division
/// inside its own proof; this is the form a sum of densities needs.
theorem upper_density_residue_class(m: Nat, a: Nat) {
    a < m implies upper_density(in_residue_class(m, a)) = Real.1 / from_nat[Real](m)
} by {
    if a < m {
        if m = Nat.0 {
            a < Nat.0
            not (a < Nat.0)
            false
        }
        m != Nat.0
        Nat.0 < m
        Nat.0.suc <= m
        (Nat.0.suc = Nat.1)
        Nat.1 <= m
        from_nat_modulus_pos(m)
        from_nat[Real](m) > Real.0
        from_nat[Real](m) != Real.0
        residue_class_density(m, a)
        (from_nat[Real](m) * upper_density(in_residue_class(m, a)) = Real.1)
        (Real.1 = from_nat[Real](m) * upper_density(in_residue_class(m, a)))
        mul_left_cancel(Real.1, from_nat[Real](m),
            upper_density(in_residue_class(m, a)))
        (Real.1 / from_nat[Real](m) = upper_density(in_residue_class(m, a)))
        upper_density(in_residue_class(m, a)) = Real.1 / from_nat[Real](m)
    }
}

/// The naturals covered by a system of residue classes.
///
/// The classes are pairs of a modulus and a residue, matching the convention of
/// `src/residue_class_disjoint.ac`, read here as predicates on the naturals.
define covers_nat(system: List[Pair[Nat, Nat]]) -> (Nat -> Bool) {
    match system {
        List.nil {
            none_pred
        }
        List.cons(head, tail) {
            or_pred(in_residue_class(head.first, head.second), covers_nat(tail))
        }
    }
}

/// The sum of the reciprocals of the moduli of a system, as a real.
///
/// `system_density` in `src/residue_class_density.ac` is the same sum over the rationals. The
/// real-valued form is what compares directly with an upper density.
define system_density_real(system: List[Pair[Nat, Nat]]) -> Real {
    match system {
        List.nil {
            Real.0
        }
        List.cons(head, tail) {
            Real.1 / from_nat[Real](head.first) + system_density_real(tail)
        }
    }
}

/// True if every class of a system has its residue below its modulus.
///
/// The normalisation the statements below assume. A residue at or above its modulus names the
/// same class as its remainder, so nothing is lost by requiring it.
define system_reduced(system: List[Pair[Nat, Nat]]) -> Bool {
    forall(cl: Pair[Nat, Nat]) {
        system.contains(cl) implies cl.second < cl.first
    }
}

/// Membership of a cons system.
theorem covers_nat_cons(head: Pair[Nat, Nat], tail: List[Pair[Nat, Nat]]) {
    covers_nat(List.cons(head, tail))
        = or_pred(in_residue_class(head.first, head.second), covers_nat(tail))
}

/// The empty system covers nothing.
theorem covers_nat_nil {
    covers_nat(List.nil[Pair[Nat, Nat]]) = none_pred
}

/// Adding a class adds the reciprocal of its modulus.
theorem system_density_real_cons(head: Pair[Nat, Nat], tail: List[Pair[Nat, Nat]]) {
    system_density_real(List.cons(head, tail))
        = Real.1 / from_nat[Real](head.first) + system_density_real(tail)
}

/// The empty system has weight zero.
theorem system_density_real_nil {
    system_density_real(List.nil[Pair[Nat, Nat]]) = Real.0
}

/// The head of a reduced system is reduced.
theorem system_reduced_head(head: Pair[Nat, Nat], tail: List[Pair[Nat, Nat]]) {
    system_reduced(List.cons(head, tail)) implies head.second < head.first
} by {
    if system_reduced(List.cons(head, tail)) {
        (system_reduced(List.cons(head, tail)) = forall(cl: Pair[Nat, Nat]) {
            List.cons(head, tail).contains(cl) implies cl.second < cl.first
        })
        List.cons(head, tail).contains(head)
        (List.cons(head, tail).contains(head) implies head.second < head.first)
        head.second < head.first
    }
}

/// The tail of a reduced system is reduced.
theorem system_reduced_tail(head: Pair[Nat, Nat], tail: List[Pair[Nat, Nat]]) {
    system_reduced(List.cons(head, tail)) implies system_reduced(tail)
} by {
    if system_reduced(List.cons(head, tail)) {
        (system_reduced(List.cons(head, tail)) = forall(cl: Pair[Nat, Nat]) {
            List.cons(head, tail).contains(cl) implies cl.second < cl.first
        })
        forall(cl: Pair[Nat, Nat]) {
            if tail.contains(cl) {
                List.cons(head, tail).contains(cl)
                (List.cons(head, tail).contains(cl) implies cl.second < cl.first)
                cl.second < cl.first
            }
            (tail.contains(cl) implies cl.second < cl.first)
        }
        (system_reduced(tail) = forall(cl: Pair[Nat, Nat]) {
            tail.contains(cl) implies cl.second < cl.first
        })
        system_reduced(tail)
    }
}

/// The upper density of the naturals a system covers is at most the sum of its weights.
///
/// Subadditivity applied along the list, with each class contributing the reciprocal of its
/// modulus. This is the inequality a covering system argument runs on: if the system covers
/// everything the left side is one, so the reciprocals sum to at least one.
theorem upper_density_covers_nat_le(system: List[Pair[Nat, Nat]]) {
    system_reduced(system)
        implies upper_density(covers_nat(system)) <= system_density_real(system)
} by {
    define p(s: List[Pair[Nat, Nat]]) -> Bool {
        system_reduced(s) implies upper_density(covers_nat(s)) <= system_density_real(s)
    }
    covers_nat_nil
    (covers_nat(List.nil[Pair[Nat, Nat]]) = none_pred)
    system_density_real_nil
    (system_density_real(List.nil[Pair[Nat, Nat]]) = Real.0)
    upper_density_none
    upper_density(none_pred) <= Real.0
    p(List.nil[Pair[Nat, Nat]])
    forall(head: Pair[Nat, Nat], tail: List[Pair[Nat, Nat]]) {
        if p(tail) {
            if system_reduced(List.cons(head, tail)) {
                system_reduced_head(head, tail)
                head.second < head.first
                system_reduced_tail(head, tail)
                system_reduced(tail)
                (system_reduced(tail) implies upper_density(covers_nat(tail))
                    <= system_density_real(tail))
                upper_density(covers_nat(tail)) <= system_density_real(tail)
                covers_nat_cons(head, tail)
                (covers_nat(List.cons(head, tail))
                    = or_pred(in_residue_class(head.first, head.second), covers_nat(tail)))
                upper_density_subadditive(
                    in_residue_class(head.first, head.second), covers_nat(tail))
                (upper_density(or_pred(in_residue_class(head.first, head.second),
                    covers_nat(tail)))
                    <= upper_density(in_residue_class(head.first, head.second))
                        + upper_density(covers_nat(tail)))
                upper_density_residue_class(head.first, head.second)
                (upper_density(in_residue_class(head.first, head.second))
                    = Real.1 / from_nat[Real](head.first))
                (upper_density(covers_nat(List.cons(head, tail)))
                    <= Real.1 / from_nat[Real](head.first)
                        + upper_density(covers_nat(tail)))
                lte_add_left(upper_density(covers_nat(tail)), system_density_real(tail),
                    Real.1 / from_nat[Real](head.first))
                (Real.1 / from_nat[Real](head.first) + upper_density(covers_nat(tail))
                    <= Real.1 / from_nat[Real](head.first) + system_density_real(tail))
                system_density_real_cons(head, tail)
                (system_density_real(List.cons(head, tail))
                    = Real.1 / from_nat[Real](head.first) + system_density_real(tail))
                lte_trans(upper_density(covers_nat(List.cons(head, tail))),
                    Real.1 / from_nat[Real](head.first) + upper_density(covers_nat(tail)),
                    system_density_real(List.cons(head, tail)))
                (upper_density(covers_nat(List.cons(head, tail)))
                    <= system_density_real(List.cons(head, tail)))
            }
            p(List.cons(head, tail))
        }
        (p(tail) implies p(List.cons(head, tail)))
    }
    p(List.nil[Pair[Nat, Nat]]) and forall(h: Pair[Nat, Nat], t: List[Pair[Nat, Nat]]) {
        p(t) implies p(List.cons(h, t))
    }
    List.induction(p)
    p(system)
}
