from nat import Nat, is_min, has_min, is_min_apply, is_min_false_below, false_below,
    false_below_apply, lte_antisymm, lt_or_lte, lte_imp_not_lt

numerals Nat

/// The pointwise negation of a predicate on the naturals.
///
/// Named so that the least natural failing a predicate can be expressed with the existing
/// minimum construction rather than a second one.
define negate_pred(p: Nat -> Bool, n: Nat) -> Bool {
    not p(n)
}

/// Negation at a point.
theorem negate_pred_eq(p: Nat -> Bool, n: Nat) {
    negate_pred(p)(n) = not p(n)
}

/// True if `m` is the smallest natural at which `p` fails.
///
/// The least-witness form that search arguments actually use: everything below `m` satisfies
/// the predicate, and `m` itself does not. Both halves are needed, and stating them together
/// keeps the two in step.
define is_least_failure(p: Nat -> Bool, m: Nat) -> Bool {
    is_min(negate_pred(p), m)
}

/// The predicate fails at the least failure.
theorem is_least_failure_fails(p: Nat -> Bool, m: Nat) {
    is_least_failure(p, m) implies not p(m)
} by {
    if is_least_failure(p, m) {
        is_least_failure(p, m) = is_min(negate_pred(p), m)
        is_min(negate_pred(p), m)
        is_min_apply(negate_pred(p), m)
        negate_pred(p)(m)
        negate_pred_eq(p, m)
        not p(m)
    }
}

/// The predicate holds strictly below the least failure.
///
/// This is the half that makes the notion useful: it converts a minimality assumption into a
/// usable fact about every smaller natural.
theorem is_least_failure_holds_below(p: Nat -> Bool, m: Nat, x: Nat) {
    is_least_failure(p, m) and x < m implies p(x)
} by {
    if is_least_failure(p, m) and x < m {
        is_least_failure(p, m) = is_min(negate_pred(p), m)
        is_min(negate_pred(p), m)
        is_min_false_below(negate_pred(p), m)
        false_below(negate_pred(p), m)
        false_below_apply(negate_pred(p), m, x)
        not negate_pred(p)(x)
        negate_pred_eq(p, x)
        p(x)
    }
}

/// The two halves give a least failure.
theorem is_least_failure_intro(p: Nat -> Bool, m: Nat) {
    not p(m) and (forall(x: Nat) { x < m implies p(x) }) implies is_least_failure(p, m)
} by {
    if not p(m) and forall(x: Nat) { x < m implies p(x) } {
        negate_pred_eq(p, m)
        negate_pred(p)(m)
        forall(x: Nat) {
            if x < m {
                p(x)
                negate_pred_eq(p, x)
                not negate_pred(p)(x)
            }
            (x < m implies not negate_pred(p)(x))
        }
        false_below(negate_pred(p), m) = forall(y: Nat) {
            y < m implies not negate_pred(p)(y)
        }
        false_below(negate_pred(p), m)
        is_min(negate_pred(p), m) = (negate_pred(p)(m) and false_below(negate_pred(p), m))
        is_min(negate_pred(p), m)
        is_least_failure(p, m) = is_min(negate_pred(p), m)
        is_least_failure(p, m)
    }
}

/// A predicate that fails anywhere has a least failure.
theorem least_failure_exists(p: Nat -> Bool, n: Nat) {
    not p(n) implies exists(m: Nat) { is_least_failure(p, m) }
} by {
    if not p(n) {
        negate_pred_eq(p, n)
        negate_pred(p)(n)
        has_min(negate_pred(p), n)
        exists(k: Nat) {
            is_min(negate_pred(p), k)
        }
        let (m: Nat) satisfy {
            is_min(negate_pred(p), m)
        }
        is_least_failure(p, m) = is_min(negate_pred(p), m)
        is_least_failure(p, m)
        exists(k: Nat) { is_least_failure(p, k) }
    }
}

/// The least failure is unique.
///
/// Each is below or equal to the other, since neither can lie strictly below a point where
/// the other says the predicate holds.
theorem least_failure_unique(p: Nat -> Bool, m: Nat, m2: Nat) {
    is_least_failure(p, m) and is_least_failure(p, m2) implies m = m2
} by {
    if is_least_failure(p, m) and is_least_failure(p, m2) {
        if m < m2 {
            is_least_failure_holds_below(p, m2, m)
            p(m)
            is_least_failure_fails(p, m)
            not p(m)
            false
        }
        not (m < m2)
        if m2 < m {
            is_least_failure_holds_below(p, m, m2)
            p(m2)
            is_least_failure_fails(p, m2)
            not p(m2)
            false
        }
        not (m2 < m)
        lt_or_lte(m, m2)
        m2 <= m
        lt_or_lte(m2, m)
        m <= m2
        lte_antisymm(m, m2)
        m = m2
    }
}

/// No failure lies below the least one.
theorem least_failure_is_least(p: Nat -> Bool, m: Nat, x: Nat) {
    is_least_failure(p, m) and not p(x) implies m <= x
} by {
    if is_least_failure(p, m) and not p(x) {
        if x < m {
            is_least_failure_holds_below(p, m, x)
            p(x)
            false
        }
        not (x < m)
        lt_or_lte(x, m)
        m <= x
    }
}

/// A least failure that is not zero means the predicate holds at zero.
theorem least_failure_positive_holds_at_zero(p: Nat -> Bool, m: Nat) {
    is_least_failure(p, m) and Nat.0 < m implies p(Nat.0)
} by {
    if is_least_failure(p, m) and Nat.0 < m {
        is_least_failure_holds_below(p, m, Nat.0)
        p(Nat.0)
    }
}

/// A predicate holding at zero has a positive least failure.
theorem least_failure_positive(p: Nat -> Bool, m: Nat) {
    is_least_failure(p, m) and p(Nat.0) implies Nat.0 < m
} by {
    if is_least_failure(p, m) and p(Nat.0) {
        if m = Nat.0 {
            is_least_failure_fails(p, m)
            not p(Nat.0)
            false
        }
        m != Nat.0
        Nat.0 < m
    }
}

/// A predicate holding below a bound has its least failure at or above that bound.
theorem least_failure_lower_bound(p: Nat -> Bool, m: Nat, b: Nat) {
    is_least_failure(p, m) and (forall(x: Nat) { x < b implies p(x) }) implies b <= m
} by {
    if is_least_failure(p, m) and forall(x: Nat) { x < b implies p(x) } {
        if m < b {
            p(m)
            is_least_failure_fails(p, m)
            not p(m)
            false
        }
        not (m < b)
        lt_or_lte(m, b)
        b <= m
    }
}
