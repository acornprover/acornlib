from nat import Nat
from algebra.semigroup import mul_fn
from semiring import Semiring
from data.nat.nat_range_sum import range_sum, range_sum_zero, range_sum_suc, range_sum_congr

numerals Nat

/// A scalar multiple passes through a range sum.
///
/// The distributive law applied once per term, which is exactly what the recurrence supplies
/// at each induction step.
theorem range_sum_scalar_mul[S: Semiring](c: S, f: Nat -> S, n: Nat) {
    c * range_sum(f, n) = range_sum(mul_fn(c, f), n)
} by {
    define p(x: Nat) -> Bool {
        c * range_sum(f, x) = range_sum(mul_fn(c, f), x)
    }
    range_sum_zero(f)
    range_sum_zero(mul_fn(c, f))
    c * S.0 = S.0
    c * range_sum(f, Nat.0) = range_sum(mul_fn(c, f), Nat.0)
    p(Nat.0)
    forall(k: Nat) {
        if p(k) {
            c * range_sum(f, k) = range_sum(mul_fn(c, f), k)
            range_sum_suc(f, k)
            range_sum(f, k.suc) = range_sum(f, k) + f(k)
            c * (range_sum(f, k) + f(k)) = c * range_sum(f, k) + c * f(k)
            mul_fn(c, f, k) = c * f(k)
            c * range_sum(f, k.suc) = range_sum(mul_fn(c, f), k) + mul_fn(c, f, k)
            range_sum_suc(mul_fn(c, f), k)
            range_sum(mul_fn(c, f), k.suc) = range_sum(mul_fn(c, f), k) + mul_fn(c, f, k)
            c * range_sum(f, k.suc) = range_sum(mul_fn(c, f), k.suc)
            p(k.suc)
        }
        (p(k) implies p(k.suc))
    }
    p(Nat.0) and forall(k: Nat) {
        p(k) implies p(k.suc)
    }
    Nat.induction(p)
    p(n)
}

/// The summand that is `c` at every index.
define const_fn[S: Semiring](c: S, i: Nat) -> S {
    c
}

/// The number of terms, as an element of the semiring.
///
/// Counted by the same recurrence the sum uses, so the two stay in step without needing a
/// homomorphism from the naturals.
define semiring_count[S: Semiring](n: Nat) -> S {
    range_sum(const_fn(S.1), n)
}

/// A constant summand sums to the count times the constant.
theorem range_sum_const[S: Semiring](c: S, n: Nat) {
    range_sum(const_fn(c), n) = semiring_count[S](n) * c
} by {
    define p(x: Nat) -> Bool {
        range_sum(const_fn(c), x) = semiring_count[S](x) * c
    }
    range_sum_zero(const_fn(c))
    range_sum_zero(const_fn(S.1))
    semiring_count[S](Nat.0) = S.0
    S.0 * c = S.0
    range_sum(const_fn(c), Nat.0) = semiring_count[S](Nat.0) * c
    p(Nat.0)
    forall(k: Nat) {
        if p(k) {
            range_sum(const_fn(c), k) = semiring_count[S](k) * c
            range_sum_suc(const_fn(c), k)
            const_fn(c, k) = c
            range_sum(const_fn(c), k.suc) = semiring_count[S](k) * c + c
            range_sum_suc(const_fn(S.1), k)
            const_fn(S.1, k) = S.1
            semiring_count[S](k.suc) = semiring_count[S](k) + S.1
            (semiring_count[S](k) + S.1) * c = semiring_count[S](k) * c + S.1 * c
            S.1 * c = c
            semiring_count[S](k.suc) * c = semiring_count[S](k) * c + c
            range_sum(const_fn(c), k.suc) = semiring_count[S](k.suc) * c
            p(k.suc)
        }
        (p(k) implies p(k.suc))
    }
    p(Nat.0) and forall(k: Nat) {
        p(k) implies p(k.suc)
    }
    Nat.induction(p)
    p(n)
}

/// The count grows by one with each extra term.
theorem semiring_count_suc[S: Semiring](n: Nat) {
    semiring_count[S](n.suc) = semiring_count[S](n) + S.1
} by {
    range_sum_suc(const_fn(S.1), n)
    const_fn(S.1, n) = S.1
    semiring_count[S](n.suc) = semiring_count[S](n) + S.1
}

/// An empty range has count zero.
theorem semiring_count_zero[S: Semiring] {
    semiring_count[S](Nat.0) = S.0
} by {
    range_sum_zero(const_fn(S.1))
}

/// A range of one has count one.
theorem semiring_count_one[S: Semiring] {
    semiring_count[S](Nat.1) = S.1
} by {
    semiring_count_suc[S](Nat.0)
    semiring_count[S](Nat.1) = semiring_count[S](Nat.0) + S.1
    semiring_count_zero[S]
    semiring_count[S](Nat.1) = S.0 + S.1
    S.0 + S.1 = S.1
}

/// A summand equal to a constant below the limit sums to the count times that constant.
///
/// The form a bound is usually applied in: the summand need only agree with the constant on
/// the range, not everywhere.
theorem range_sum_const_congr[S: Semiring](f: Nat -> S, c: S, n: Nat) {
    (forall(i: Nat) { i < n implies f(i) = c })
        implies range_sum(f, n) = semiring_count[S](n) * c
} by {
    if forall(i: Nat) { i < n implies f(i) = c } {
        forall(i: Nat) {
            const_fn(c, i) = c
            (i < n implies f(i) = const_fn(c, i))
        }
        range_sum_congr(f, const_fn(c), n)
        range_sum(f, n) = range_sum(const_fn(c), n)
        range_sum_const(c, n)
        range_sum(f, n) = semiring_count[S](n) * c
    }
}

/// The zero constant sums to zero.
theorem range_sum_const_zero[S: Semiring](n: Nat) {
    range_sum(const_fn(S.0), n) = S.0
} by {
    range_sum_const(S.0, n)
    range_sum(const_fn(S.0), n) = semiring_count[S](n) * S.0
    semiring_count[S](n) * S.0 = S.0
}
