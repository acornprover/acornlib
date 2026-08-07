from nat import Nat, pow_zero, pow_one, pow_add, lt_and_lte, mul_cancel_left,
    lte_mul_both, lt_mul_both, lte_trans
from combinatorics import binom, choose_zero, choose_out_of_bounds, choose_symm_add_form,
    pascal_suc, binom_complement_absorption
from number_theory import central_binom, central_binom_eq, central_binom_zero

numerals Nat

/// Just past the middle of an even row, the binomial coefficient does not grow.
///
/// The complementary absorption identity at the middle: `n` times the central coefficient is
/// `n + 1` times the next one, and `n` is the smaller multiplier, so the next one is the smaller
/// coefficient. This is the only piece of unimodality the bound below needs.
theorem binom_next_to_central_lte(n: Nat) {
    (n + n).binom(n.suc) <= (n + n).binom(n)
} by {
    if n = Nat.0 {
        (n + n = Nat.0)
        (n.suc = Nat.1)
        Nat.0 < Nat.1
        choose_out_of_bounds(Nat.0, Nat.1)
        (Nat.0.binom(Nat.1) = Nat.0)
        ((n + n).binom(n.suc) = Nat.0)
        (Nat.0 <= (n + n).binom(n))
        (n + n).binom(n.suc) <= (n + n).binom(n)
    }
    if n != Nat.0 {
        Nat.0 < n
        (n < n + n)
        binom_complement_absorption(n + n, n)
        ((n + n - n) * (n + n).binom(n) = n.suc * (n + n).binom(n.suc))
        (n + n - n = n)
        (n * (n + n).binom(n) = n.suc * (n + n).binom(n.suc))
        (n <= n.suc)
        lte_mul_both((n + n).binom(n), n, n.suc)
        (n * (n + n).binom(n) <= n.suc * (n + n).binom(n))
        (n.suc * (n + n).binom(n.suc) <= n.suc * (n + n).binom(n))
        Nat.0 < n.suc
        n.suc != Nat.0
        if (n + n).binom(n) < (n + n).binom(n.suc) {
            lt_mul_both(n.suc, (n + n).binom(n), (n + n).binom(n.suc))
            (n.suc * (n + n).binom(n) < n.suc * (n + n).binom(n.suc))
            false
        }
        (n + n).binom(n.suc) <= (n + n).binom(n)
    }
    (n + n).binom(n.suc) <= (n + n).binom(n)
}

/// Each central binomial coefficient is at most four times the one before it.
///
/// Pascal twice and the symmetry of an odd row: the coefficient at the middle of row `2n + 2` is
/// twice the one just left of the middle of row `2n + 1`, which is the sum of the central
/// coefficient of row `2n` and the one just past it, and that one is no larger.
theorem central_binom_suc_lte(n: Nat) {
    central_binom(n.suc) <= Nat.4 * central_binom(n)
} by {
    central_binom_eq(n.suc)
    (central_binom(n.suc) = (n.suc + n.suc).binom(n.suc))
    (n.suc + n.suc = (n + n.suc).suc)
    (n <= n + n.suc)
    pascal_suc(n + n.suc, n)
    ((n + n.suc).suc.binom(n.suc) = (n + n.suc).binom(n) + (n + n.suc).binom(n.suc))
    choose_symm_add_form(n, n.suc)
    ((n + n.suc).binom(n) = (n + n.suc).binom(n.suc))
    (central_binom(n.suc) = (n + n.suc).binom(n.suc) + (n + n.suc).binom(n.suc))
    (n <= n + n)
    pascal_suc(n + n, n)
    ((n + n).suc.binom(n.suc) = (n + n).binom(n) + (n + n).binom(n.suc))
    (n + n.suc = (n + n).suc)
    ((n + n.suc).binom(n.suc) = (n + n).binom(n) + (n + n).binom(n.suc))
    binom_next_to_central_lte(n)
    ((n + n).binom(n.suc) <= (n + n).binom(n))
    ((n + n).binom(n) + (n + n).binom(n.suc)
        <= (n + n).binom(n) + (n + n).binom(n))
    central_binom_eq(n)
    (central_binom(n) = (n + n).binom(n))
    ((n + n.suc).binom(n.suc) <= central_binom(n) + central_binom(n))
    ((n + n.suc).binom(n.suc) + (n + n.suc).binom(n.suc)
        <= (central_binom(n) + central_binom(n)) + (central_binom(n) + central_binom(n)))
    (central_binom(n) + central_binom(n) = Nat.2 * central_binom(n))
    (Nat.2 * central_binom(n) + Nat.2 * central_binom(n)
        = Nat.2 * (Nat.2 * central_binom(n)))
    (Nat.2 * (Nat.2 * central_binom(n)) = (Nat.2 * Nat.2) * central_binom(n))
    (Nat.2 * Nat.2 = Nat.4)
    ((central_binom(n) + central_binom(n)) + (central_binom(n) + central_binom(n))
        = Nat.4 * central_binom(n))
    central_binom(n.suc) <= Nat.4 * central_binom(n)
}

/// The central binomial coefficient is at most four to the power of its index.
///
/// The size estimate a Bertrand-style argument needs on the other side of the product bound.
theorem central_binom_lte_four_pow(n: Nat) {
    central_binom(n) <= Nat.4.pow(n)
} by {
    define p(k: Nat) -> Bool {
        central_binom(k) <= Nat.4.pow(k)
    }
    central_binom_zero
    (central_binom(Nat.0) = Nat.1)
    pow_zero[Nat](Nat.4)
    (Nat.4.pow(Nat.0) = Nat.1)
    (Nat.1 <= Nat.1)
    (central_binom(Nat.0) <= Nat.4.pow(Nat.0))
    p(Nat.0)
    forall(k: Nat) {
        if p(k) {
            (central_binom(k) <= Nat.4.pow(k))
            central_binom_suc_lte(k)
            (central_binom(k.suc) <= Nat.4 * central_binom(k))
            lte_mul_both(Nat.4, central_binom(k), Nat.4.pow(k))
            (Nat.4 * central_binom(k) <= Nat.4 * Nat.4.pow(k))
            pow_one[Nat](Nat.4)
            (Nat.4.pow(Nat.1) = Nat.4)
            pow_add[Nat](Nat.4, Nat.1, k)
            (Nat.4.pow(Nat.1 + k) = Nat.4.pow(Nat.1) * Nat.4.pow(k))
            (Nat.1 + k = k.suc)
            (Nat.4.pow(k.suc) = Nat.4 * Nat.4.pow(k))
            lte_trans(central_binom(k.suc), Nat.4 * central_binom(k), Nat.4.pow(k.suc))
            central_binom(k.suc) <= Nat.4.pow(k.suc)
            p(k.suc)
        }
        (p(k) implies p(k.suc))
    }
    p(Nat.0) and forall(k: Nat) {
        p(k) implies p(k.suc)
    }
    Nat.induction(p)
    p(n)
}
