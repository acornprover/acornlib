from nat import Nat, mod_of_decomp, mod_lt, small_mod, lte_and_lt, lt_and_lte, lte_antisymm
from data.nat.nat_counting_function import count_upto, count_upto_zero, count_upto_suc_true,
    count_upto_suc_false

numerals Nat

/// True if a natural leaves remainder `a` on division by `m`.
///
/// Packaged as a predicate on `Nat` so the counting function applies to it, which is what a
/// density statement about a residue class needs.
define in_residue_class(m: Nat, a: Nat) -> (Nat -> Bool) {
    function(k: Nat) {
        k.mod(m) = a
    }
}

/// Membership in a residue class is the remainder condition.
theorem in_residue_class_apply(m: Nat, a: Nat, k: Nat) {
    in_residue_class(m, a)(k) = (k.mod(m) = a)
} by {
    in_residue_class(m, a)(k) = (k.mod(m) = a)
}

/// Within one block, a residue class is hit exactly at the offset equal to the residue.
theorem residue_class_in_block(m: Nat, a: Nat, q: Nat, i: Nat) {
    a < m and i < m implies (in_residue_class(m, a)(q * m + i) = (i = a))
} by {
    if a < m and i < m {
        mod_of_decomp(q, i, m)
        (q * m + i).mod(m) = i
        in_residue_class_apply(m, a, q * m + i)
        (in_residue_class(m, a)(q * m + i) = ((q * m + i).mod(m) = a))
        (in_residue_class(m, a)(q * m + i) = (i = a))
    }
}

/// Before the residue is reached, walking into a block adds nothing to the count.
///
/// Split from the case below rather than written with a conditional term. A term of the form
/// `if a < j { 1 } else { 0 }` sends proof search into a blowup here — the module did not
/// verify in twenty minutes with it and verifies quickly without it.
theorem count_upto_in_block_before(m: Nat, a: Nat, q: Nat, j: Nat) {
    a < m and j <= a implies count_upto(in_residue_class(m, a), q * m + j)
        = count_upto(in_residue_class(m, a), q * m)
} by {
    if a < m {
        define p(x: Nat) -> Bool {
            x <= a implies count_upto(in_residue_class(m, a), q * m + x)
                = count_upto(in_residue_class(m, a), q * m)
        }
        (q * m + Nat.0 = q * m)
        p(Nat.0)
        forall(k: Nat) {
            if p(k) {
                if k.suc <= a {
                    k < k.suc
                    lt_and_lte(k, k.suc, a)
                    k < a
                    k <= a
                    (k <= a implies count_upto(in_residue_class(m, a), q * m + k)
                        = count_upto(in_residue_class(m, a), q * m))
                    (count_upto(in_residue_class(m, a), q * m + k)
                        = count_upto(in_residue_class(m, a), q * m))
                    lt_and_lte(k, a, m)
                    k < m
                    residue_class_in_block(m, a, q, k)
                    (in_residue_class(m, a)(q * m + k) = (k = a))
                    k != a
                    not in_residue_class(m, a)(q * m + k)
                    count_upto_suc_false(in_residue_class(m, a), q * m + k)
                    (count_upto(in_residue_class(m, a), (q * m + k).suc)
                        = count_upto(in_residue_class(m, a), q * m + k))
                    (q * m + k.suc = (q * m + k).suc)
                    (count_upto(in_residue_class(m, a), q * m + k.suc)
                        = count_upto(in_residue_class(m, a), q * m))
                }
                p(k.suc)
            }
            (p(k) implies p(k.suc))
        }
        p(Nat.0) and forall(k: Nat) {
            p(k) implies p(k.suc)
        }
        Nat.induction(p)
        p(j)
        (j <= a implies count_upto(in_residue_class(m, a), q * m + j)
            = count_upto(in_residue_class(m, a), q * m))
    }
}

/// Once the residue is passed, walking into a block has added exactly one.
theorem count_upto_in_block_after(m: Nat, a: Nat, q: Nat, j: Nat) {
    a < m and a < j and j <= m implies count_upto(in_residue_class(m, a), q * m + j)
        = count_upto(in_residue_class(m, a), q * m) + Nat.1
} by {
    if a < m {
        define p(x: Nat) -> Bool {
            a < x and x <= m implies count_upto(in_residue_class(m, a), q * m + x)
                = count_upto(in_residue_class(m, a), q * m) + Nat.1
        }
        not (a < Nat.0)
        p(Nat.0)
        forall(k: Nat) {
            if p(k) {
                if a < k.suc and k.suc <= m {
                    k < k.suc
                    lt_and_lte(k, k.suc, m)
                    k < m
                    (q * m + k.suc = (q * m + k).suc)
                    residue_class_in_block(m, a, q, k)
                    (in_residue_class(m, a)(q * m + k) = (k = a))
                    if k = a {
                        in_residue_class(m, a)(q * m + k)
                        count_upto_suc_true(in_residue_class(m, a), q * m + k)
                        (count_upto(in_residue_class(m, a), (q * m + k).suc)
                            = count_upto(in_residue_class(m, a), q * m + k) + Nat.1)
                        a <= a
                        count_upto_in_block_before(m, a, q, k)
                        (count_upto(in_residue_class(m, a), q * m + k)
                            = count_upto(in_residue_class(m, a), q * m))
                        (count_upto(in_residue_class(m, a), q * m + k.suc)
                            = count_upto(in_residue_class(m, a), q * m) + Nat.1)
                    }
                    if k != a {
                        a <= k
                        (a < k or a = k)
                        a < k
                        k <= m
                        (a < k and k <= m implies
                            count_upto(in_residue_class(m, a), q * m + k)
                                = count_upto(in_residue_class(m, a), q * m) + Nat.1)
                        (count_upto(in_residue_class(m, a), q * m + k)
                            = count_upto(in_residue_class(m, a), q * m) + Nat.1)
                        not in_residue_class(m, a)(q * m + k)
                        count_upto_suc_false(in_residue_class(m, a), q * m + k)
                        (count_upto(in_residue_class(m, a), (q * m + k).suc)
                            = count_upto(in_residue_class(m, a), q * m + k))
                        (count_upto(in_residue_class(m, a), q * m + k.suc)
                            = count_upto(in_residue_class(m, a), q * m) + Nat.1)
                    }
                    (count_upto(in_residue_class(m, a), q * m + k.suc)
                        = count_upto(in_residue_class(m, a), q * m) + Nat.1)
                }
                p(k.suc)
            }
            (p(k) implies p(k.suc))
        }
        p(Nat.0) and forall(k: Nat) {
            p(k) implies p(k.suc)
        }
        Nat.induction(p)
        p(j)
        (a < j and j <= m implies count_upto(in_residue_class(m, a), q * m + j)
            = count_upto(in_residue_class(m, a), q * m) + Nat.1)
    }
}

/// Crossing one whole block adds exactly one to the count.
theorem count_upto_block_step(m: Nat, a: Nat, q: Nat) {
    a < m implies count_upto(in_residue_class(m, a), q * m + m)
        = count_upto(in_residue_class(m, a), q * m) + Nat.1
} by {
    if a < m {
        m <= m
        count_upto_in_block_after(m, a, q, m)
        (count_upto(in_residue_class(m, a), q * m + m)
            = count_upto(in_residue_class(m, a), q * m) + Nat.1)
    }
}

/// A residue class is hit exactly once per block.
///
/// The exact count over a whole number of periods, which is what makes the density of a residue
/// class the reciprocal of its modulus.
theorem count_upto_multiple(m: Nat, a: Nat, q: Nat) {
    a < m implies count_upto(in_residue_class(m, a), q * m) = q
} by {
    if a < m {
        define p(x: Nat) -> Bool {
            count_upto(in_residue_class(m, a), x * m) = x
        }
        (Nat.0 * m = Nat.0)
        count_upto_zero(in_residue_class(m, a))
        count_upto(in_residue_class(m, a), Nat.0) = Nat.0
        p(Nat.0)
        forall(k: Nat) {
            if p(k) {
                count_upto(in_residue_class(m, a), k * m) = k
                count_upto_block_step(m, a, k)
                (count_upto(in_residue_class(m, a), k * m + m)
                    = count_upto(in_residue_class(m, a), k * m) + Nat.1)
                (count_upto(in_residue_class(m, a), k * m + m) = k + Nat.1)
                (k.suc * m = k * m + m)
                (k + Nat.1 = k.suc)
                count_upto(in_residue_class(m, a), k.suc * m) = k.suc
                p(k.suc)
            }
            (p(k) implies p(k.suc))
        }
        p(Nat.0) and forall(k: Nat) {
            p(k) implies p(k.suc)
        }
        Nat.induction(p)
        p(q)
        count_upto(in_residue_class(m, a), q * m) = q
    }
}
