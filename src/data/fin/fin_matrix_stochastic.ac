from nat import Nat
from data.fin.fin import Fin, fin_zero
from semiring import Semiring
from ordered_field import OrderedField
from data.fin.fin_sum import fin_sum, fin_sum_pointwise_eq
from data.fin.fin_sum_enum import fin_sum_single_nonzero
from data.fin.fin_sum_order import fin_sum_term_le, fin_sum_nonneg, fin_sum_mono
from data.fin.fin_matrix import Matrix, matrix_entry, matrix_one, matrix_one_entry,
    matrix_transpose, matrix_entry_transpose
from data.fin.fin_matrix_eigen import matrix_apply, matrix_apply_term, matrix_apply_eq,
    matrix_row_fn, matrix_row_sum, is_row_stochastic, is_row_stochastic_apply,
    is_row_stochastic_intro, is_nonzero_vector, is_nonzero_vector_intro,
    is_nonzero_vector_witness, is_eigenvector, is_eigenvector_intro, is_eigenvector_apply,
    is_eigenvalue, is_eigenvalue_intro, ones_vector, row_stochastic_has_eigenvalue_one

numerals Nat

/// True if every entry of the matrix is nonnegative.
///
/// Separate from row-stochasticity, which needs no order. A matrix that is both is stochastic
/// in the usual sense.
define is_nonnegative_matrix[S: OrderedField](n: Nat, a: Matrix[S, n, n]) -> Bool {
    forall(i: Fin[n], j: Fin[n]) {
        S.0 <= matrix_entry[S](n, n, a, i, j)
    }
}

/// Each entry of such a matrix is nonnegative.
theorem is_nonnegative_matrix_apply[S: OrderedField](
    n: Nat, a: Matrix[S, n, n], i: Fin[n], j: Fin[n]
) {
    is_nonnegative_matrix[S](n, a) implies S.0 <= matrix_entry[S](n, n, a, i, j)
} by {
    if is_nonnegative_matrix[S](n, a) {
        is_nonnegative_matrix[S](n, a) = forall(k: Fin[n], l: Fin[n]) {
            S.0 <= matrix_entry[S](n, n, a, k, l)
        }
        forall(k: Fin[n], l: Fin[n]) {
            S.0 <= matrix_entry[S](n, n, a, k, l)
        }
        S.0 <= matrix_entry[S](n, n, a, i, j)
    }
}

/// The pointwise condition on entries is nonnegativity.
theorem is_nonnegative_matrix_intro[S: OrderedField](n: Nat, a: Matrix[S, n, n]) {
    (forall(i: Fin[n], j: Fin[n]) { S.0 <= matrix_entry[S](n, n, a, i, j) })
        implies is_nonnegative_matrix[S](n, a)
} by {
    if forall(i: Fin[n], j: Fin[n]) { S.0 <= matrix_entry[S](n, n, a, i, j) } {
        is_nonnegative_matrix[S](n, a) = forall(k: Fin[n], l: Fin[n]) {
            S.0 <= matrix_entry[S](n, n, a, k, l)
        }
        is_nonnegative_matrix[S](n, a)
    }
}

/// True if the matrix is nonnegative and every row sums to one.
///
/// The stochastic condition in full. The two halves are kept separate above because the row
/// sum condition makes sense over any semiring while nonnegativity needs an order.
define is_stochastic[S: OrderedField](n: Nat, a: Matrix[S, n, n]) -> Bool {
    is_nonnegative_matrix[S](n, a) and is_row_stochastic[S](n, a)
}

/// A stochastic matrix is nonnegative.
theorem is_stochastic_nonnegative[S: OrderedField](n: Nat, a: Matrix[S, n, n]) {
    is_stochastic[S](n, a) implies is_nonnegative_matrix[S](n, a)
} by {
    if is_stochastic[S](n, a) {
        is_stochastic[S](n, a) =
            (is_nonnegative_matrix[S](n, a) and is_row_stochastic[S](n, a))
        is_nonnegative_matrix[S](n, a)
    }
}

/// A stochastic matrix has unit row sums.
theorem is_stochastic_row_stochastic[S: OrderedField](n: Nat, a: Matrix[S, n, n]) {
    is_stochastic[S](n, a) implies is_row_stochastic[S](n, a)
} by {
    if is_stochastic[S](n, a) {
        is_stochastic[S](n, a) =
            (is_nonnegative_matrix[S](n, a) and is_row_stochastic[S](n, a))
        is_row_stochastic[S](n, a)
    }
}

/// The two conditions give a stochastic matrix.
theorem is_stochastic_intro[S: OrderedField](n: Nat, a: Matrix[S, n, n]) {
    is_nonnegative_matrix[S](n, a) and is_row_stochastic[S](n, a)
        implies is_stochastic[S](n, a)
} by {
    if is_nonnegative_matrix[S](n, a) and is_row_stochastic[S](n, a) {
        is_stochastic[S](n, a) =
            (is_nonnegative_matrix[S](n, a) and is_row_stochastic[S](n, a))
        is_stochastic[S](n, a)
    }
}

/// Every entry of a stochastic matrix is at most one.
///
/// The entry is one term of a row sum whose terms are all nonnegative and which totals one,
/// so it cannot exceed the total. This needs `fin_sum_term_le`, which is why the order lemmas
/// in `src/fin_sum_order.ac` were added.
theorem stochastic_entry_le_one[S: OrderedField](
    n: Nat, a: Matrix[S, n, n], i: Fin[n], j: Fin[n]
) {
    is_stochastic[S](n, a) implies matrix_entry[S](n, n, a, i, j) <= S.1
} by {
    if is_stochastic[S](n, a) {
        is_stochastic_nonnegative[S](n, a)
        is_nonnegative_matrix[S](n, a)
        forall(k: Fin[n]) {
            is_nonnegative_matrix_apply[S](n, a, i, k)
            S.0 <= matrix_entry[S](n, n, a, i, k)
            matrix_row_fn[S](n, a, i, k) = matrix_entry[S](n, n, a, i, k)
            S.0 <= matrix_row_fn[S](n, a, i, k)
        }
        fin_sum_term_le[S](n, matrix_row_fn[S](n, a, i), j)
        matrix_row_fn[S](n, a, i, j) <= fin_sum[S](n, matrix_row_fn[S](n, a, i))
        matrix_row_fn[S](n, a, i, j) = matrix_entry[S](n, n, a, i, j)
        matrix_row_sum[S](n, a, i) = fin_sum[S](n, matrix_row_fn[S](n, a, i))
        matrix_entry[S](n, n, a, i, j) <= matrix_row_sum[S](n, a, i)
        is_stochastic_row_stochastic[S](n, a)
        is_row_stochastic[S](n, a)
        is_row_stochastic_apply[S](n, a, i)
        matrix_row_sum[S](n, a, i) = S.1
        matrix_entry[S](n, n, a, i, j) <= S.1
    }
}

/// The row sums of the identity matrix are one.
///
/// Exactly one entry of each row is one and the rest are zero.
theorem matrix_one_row_sum[S: Semiring](n: Nat, i: Fin[n]) {
    matrix_row_sum[S](n, matrix_one[S](n), i) = S.1
} by {
    forall(k: Fin[n]) {
        matrix_row_fn[S](n, matrix_one[S](n), i, k) = matrix_entry[S](n, n, matrix_one[S](n), i, k)
        matrix_entry[S](n, n, matrix_one[S](n), i, k) = matrix_one_entry[S](n, i, k)
        matrix_row_fn[S](n, matrix_one[S](n), i, k) = matrix_one_entry[S](n, i, k)
    }
    fin_sum_pointwise_eq[S](n, matrix_row_fn[S](n, matrix_one[S](n), i), matrix_one_entry[S](n, i))
    (fin_sum[S](n, matrix_row_fn[S](n, matrix_one[S](n), i))
        = fin_sum[S](n, matrix_one_entry[S](n, i)))
    forall(k: Fin[n]) {
        if k != i {
            matrix_one_entry[S](n, i, k) = S.0
        }
        (k != i implies matrix_one_entry[S](n, i, k) = S.0)
    }
    fin_sum_single_nonzero[S](n, matrix_one_entry[S](n, i), i)
    fin_sum[S](n, matrix_one_entry[S](n, i)) = matrix_one_entry[S](n, i, i)
    matrix_one_entry[S](n, i, i) = S.1
    fin_sum[S](n, matrix_one_entry[S](n, i)) = S.1
    matrix_row_sum[S](n, matrix_one[S](n), i) = S.1
}

/// The identity matrix is row-stochastic.
theorem matrix_one_is_row_stochastic[S: Semiring](n: Nat) {
    is_row_stochastic[S](n, matrix_one[S](n))
} by {
    forall(i: Fin[n]) {
        matrix_one_row_sum[S](n, i)
        matrix_row_sum[S](n, matrix_one[S](n), i) = S.1
    }
    is_row_stochastic_intro[S](n, matrix_one[S](n))
    is_row_stochastic[S](n, matrix_one[S](n))
}

/// The `k`th entry of a column of the matrix.
define matrix_col_fn[S: Semiring](
    n: Nat, a: Matrix[S, n, n], j: Fin[n], k: Fin[n]
) -> S {
    matrix_entry[S](n, n, a, k, j)
}

/// The sum of the entries in a column.
define matrix_col_sum[S: Semiring](n: Nat, a: Matrix[S, n, n], j: Fin[n]) -> S {
    fin_sum[S](n, matrix_col_fn[S](n, a, j))
}

/// The column sums of a matrix are the row sums of its transpose.
///
/// Transposing exchanges the two indices, so the two sums run over the same terms.
theorem matrix_col_sum_transpose[S: Semiring](n: Nat, a: Matrix[S, n, n], j: Fin[n]) {
    matrix_col_sum[S](n, a, j) = matrix_row_sum[S](n, matrix_transpose[S](n, n, a), j)
} by {
    forall(k: Fin[n]) {
        matrix_entry_transpose[S](n, n, a, k, j)
        (matrix_entry[S](n, n, matrix_transpose[S](n, n, a), j, k)
            = matrix_entry[S](n, n, a, k, j))
        matrix_col_fn[S](n, a, j, k) = matrix_entry[S](n, n, a, k, j)
        (matrix_row_fn[S](n, matrix_transpose[S](n, n, a), j, k)
            = matrix_entry[S](n, n, matrix_transpose[S](n, n, a), j, k))
        (matrix_col_fn[S](n, a, j, k)
            = matrix_row_fn[S](n, matrix_transpose[S](n, n, a), j, k))
    }
    fin_sum_pointwise_eq[S](n, matrix_col_fn[S](n, a, j),
        matrix_row_fn[S](n, matrix_transpose[S](n, n, a), j))
    (fin_sum[S](n, matrix_col_fn[S](n, a, j))
        = fin_sum[S](n, matrix_row_fn[S](n, matrix_transpose[S](n, n, a), j)))
    matrix_col_sum[S](n, a, j) = matrix_row_sum[S](n, matrix_transpose[S](n, n, a), j)
}

/// True if every column of the matrix sums to one.
define is_col_stochastic[S: Semiring](n: Nat, a: Matrix[S, n, n]) -> Bool {
    forall(j: Fin[n]) {
        matrix_col_sum[S](n, a, j) = S.1
    }
}

/// Each column of such a matrix sums to one.
theorem is_col_stochastic_apply[S: Semiring](n: Nat, a: Matrix[S, n, n], j: Fin[n]) {
    is_col_stochastic[S](n, a) implies matrix_col_sum[S](n, a, j) = S.1
} by {
    if is_col_stochastic[S](n, a) {
        is_col_stochastic[S](n, a) = forall(k: Fin[n]) {
            matrix_col_sum[S](n, a, k) = S.1
        }
        forall(k: Fin[n]) {
            matrix_col_sum[S](n, a, k) = S.1
        }
        matrix_col_sum[S](n, a, j) = S.1
    }
}

/// The pointwise condition on columns is column-stochasticity.
theorem is_col_stochastic_intro[S: Semiring](n: Nat, a: Matrix[S, n, n]) {
    (forall(j: Fin[n]) { matrix_col_sum[S](n, a, j) = S.1 })
        implies is_col_stochastic[S](n, a)
} by {
    if forall(j: Fin[n]) { matrix_col_sum[S](n, a, j) = S.1 } {
        is_col_stochastic[S](n, a) = forall(k: Fin[n]) {
            matrix_col_sum[S](n, a, k) = S.1
        }
        is_col_stochastic[S](n, a)
    }
}

/// A matrix is column-stochastic exactly when its transpose is row-stochastic.
theorem col_stochastic_iff_transpose_row_stochastic[S: Semiring](
    n: Nat, a: Matrix[S, n, n]
) {
    is_col_stochastic[S](n, a) = is_row_stochastic[S](n, matrix_transpose[S](n, n, a))
} by {
    if is_col_stochastic[S](n, a) {
        forall(j: Fin[n]) {
            is_col_stochastic_apply[S](n, a, j)
            matrix_col_sum[S](n, a, j) = S.1
            matrix_col_sum_transpose[S](n, a, j)
            matrix_row_sum[S](n, matrix_transpose[S](n, n, a), j) = S.1
        }
        is_row_stochastic_intro[S](n, matrix_transpose[S](n, n, a))
        is_row_stochastic[S](n, matrix_transpose[S](n, n, a))
    }
    if is_row_stochastic[S](n, matrix_transpose[S](n, n, a)) {
        forall(j: Fin[n]) {
            is_row_stochastic_apply[S](n, matrix_transpose[S](n, n, a), j)
            matrix_row_sum[S](n, matrix_transpose[S](n, n, a), j) = S.1
            matrix_col_sum_transpose[S](n, a, j)
            matrix_col_sum[S](n, a, j) = S.1
        }
        is_col_stochastic_intro[S](n, a)
        is_col_stochastic[S](n, a)
    }
    (is_col_stochastic[S](n, a) implies is_row_stochastic[S](n, matrix_transpose[S](n, n, a)))
    (is_row_stochastic[S](n, matrix_transpose[S](n, n, a)) implies is_col_stochastic[S](n, a))
    is_col_stochastic[S](n, a) = is_row_stochastic[S](n, matrix_transpose[S](n, n, a))
}

/// True if the matrix is stochastic in both directions.
define is_doubly_stochastic[S: OrderedField](n: Nat, a: Matrix[S, n, n]) -> Bool {
    is_stochastic[S](n, a) and is_col_stochastic[S](n, a)
}

/// A doubly stochastic matrix is stochastic.
theorem is_doubly_stochastic_row[S: OrderedField](n: Nat, a: Matrix[S, n, n]) {
    is_doubly_stochastic[S](n, a) implies is_stochastic[S](n, a)
} by {
    if is_doubly_stochastic[S](n, a) {
        is_doubly_stochastic[S](n, a) =
            (is_stochastic[S](n, a) and is_col_stochastic[S](n, a))
        is_stochastic[S](n, a)
    }
}

/// A doubly stochastic matrix is column-stochastic.
theorem is_doubly_stochastic_col[S: OrderedField](n: Nat, a: Matrix[S, n, n]) {
    is_doubly_stochastic[S](n, a) implies is_col_stochastic[S](n, a)
} by {
    if is_doubly_stochastic[S](n, a) {
        is_doubly_stochastic[S](n, a) =
            (is_stochastic[S](n, a) and is_col_stochastic[S](n, a))
        is_col_stochastic[S](n, a)
    }
}

/// The identity matrix has one as an eigenvalue.
///
/// It is row-stochastic, and every row-stochastic matrix scales the all-ones vector by one.
theorem matrix_one_has_eigenvalue_one[S: Semiring](n: Nat) {
    S.1 != S.0 implies is_eigenvalue[S](n.suc, matrix_one[S](n.suc), S.1)
} by {
    if S.1 != S.0 {
        matrix_one_is_row_stochastic[S](n.suc)
        is_row_stochastic[S](n.suc, matrix_one[S](n.suc))
        row_stochastic_has_eigenvalue_one[S](n, matrix_one[S](n.suc))
        is_eigenvalue[S](n.suc, matrix_one[S](n.suc), S.1)
    }
}

/// Every row sum of a nonnegative matrix is nonnegative.
theorem nonnegative_row_sum_nonneg[S: OrderedField](
    n: Nat, a: Matrix[S, n, n], i: Fin[n]
) {
    is_nonnegative_matrix[S](n, a) implies S.0 <= matrix_row_sum[S](n, a, i)
} by {
    if is_nonnegative_matrix[S](n, a) {
        forall(k: Fin[n]) {
            is_nonnegative_matrix_apply[S](n, a, i, k)
            S.0 <= matrix_entry[S](n, n, a, i, k)
            matrix_row_fn[S](n, a, i, k) = matrix_entry[S](n, n, a, i, k)
            S.0 <= matrix_row_fn[S](n, a, i, k)
        }
        fin_sum_nonneg[S](n, matrix_row_fn[S](n, a, i))
        S.0 <= fin_sum[S](n, matrix_row_fn[S](n, a, i))
        matrix_row_sum[S](n, a, i) = fin_sum[S](n, matrix_row_fn[S](n, a, i))
        S.0 <= matrix_row_sum[S](n, a, i)
    }
}

/// Every column sum of a nonnegative matrix is nonnegative.
theorem nonnegative_col_sum_nonneg[S: OrderedField](
    n: Nat, a: Matrix[S, n, n], j: Fin[n]
) {
    is_nonnegative_matrix[S](n, a) implies S.0 <= matrix_col_sum[S](n, a, j)
} by {
    if is_nonnegative_matrix[S](n, a) {
        forall(k: Fin[n]) {
            is_nonnegative_matrix_apply[S](n, a, k, j)
            S.0 <= matrix_entry[S](n, n, a, k, j)
            matrix_col_fn[S](n, a, j, k) = matrix_entry[S](n, n, a, k, j)
            S.0 <= matrix_col_fn[S](n, a, j, k)
        }
        fin_sum_nonneg[S](n, matrix_col_fn[S](n, a, j))
        S.0 <= fin_sum[S](n, matrix_col_fn[S](n, a, j))
        matrix_col_sum[S](n, a, j) = fin_sum[S](n, matrix_col_fn[S](n, a, j))
        S.0 <= matrix_col_sum[S](n, a, j)
    }
}

/// Every entry of a doubly stochastic matrix is at most one in its column too.
///
/// The column sums are also one, so the same term bound applies down a column.
theorem doubly_stochastic_entry_le_one[S: OrderedField](
    n: Nat, a: Matrix[S, n, n], i: Fin[n], j: Fin[n]
) {
    is_doubly_stochastic[S](n, a) implies matrix_entry[S](n, n, a, i, j) <= S.1
} by {
    if is_doubly_stochastic[S](n, a) {
        is_doubly_stochastic_row[S](n, a)
        is_stochastic[S](n, a)
        stochastic_entry_le_one[S](n, a, i, j)
        matrix_entry[S](n, n, a, i, j) <= S.1
    }
}

/// A row sum of a matrix dominated entrywise by another is at most the other's.
theorem row_sum_mono[S: OrderedField](
    n: Nat, a: Matrix[S, n, n], b: Matrix[S, n, n], i: Fin[n]
) {
    (forall(k: Fin[n]) {
        matrix_entry[S](n, n, a, i, k) <= matrix_entry[S](n, n, b, i, k)
    }) implies matrix_row_sum[S](n, a, i) <= matrix_row_sum[S](n, b, i)
} by {
    if forall(k: Fin[n]) {
        matrix_entry[S](n, n, a, i, k) <= matrix_entry[S](n, n, b, i, k)
    } {
        forall(k: Fin[n]) {
            matrix_row_fn[S](n, a, i, k) = matrix_entry[S](n, n, a, i, k)
            matrix_row_fn[S](n, b, i, k) = matrix_entry[S](n, n, b, i, k)
            matrix_row_fn[S](n, a, i, k) <= matrix_row_fn[S](n, b, i, k)
        }
        fin_sum_mono[S](n, matrix_row_fn[S](n, a, i), matrix_row_fn[S](n, b, i))
        (fin_sum[S](n, matrix_row_fn[S](n, a, i))
            <= fin_sum[S](n, matrix_row_fn[S](n, b, i)))
        matrix_row_sum[S](n, a, i) = fin_sum[S](n, matrix_row_fn[S](n, a, i))
        matrix_row_sum[S](n, b, i) = fin_sum[S](n, matrix_row_fn[S](n, b, i))
        matrix_row_sum[S](n, a, i) <= matrix_row_sum[S](n, b, i)
    }
}
