from nat import Nat, strong_induction, true_below, true_below_apply, lt_and_lte,
    lte_and_lt, lte_cancel_suc, lt_cancel_suc, lt_suc, lt_not_ref, lte_antisymm, lt_add_left, lt_trans,
    add_suc_left, trichotomy
from data.fin.fin import Fin, fin_zero, fin_zero_new, fin_zero_value, fin_of_nat_suc,
    fin_of_nat_suc_new_of_lt, fin_last, fin_last_new, fin_last_value, fin_succ, fin_succ_value,
    ext
from data.fin.fin_enum import fin_enum_suc, fin_enum_suc_contains,
    fin_enum_suc_is_unique, fin_enum_suc_length
from data.fin.fin_matrix import Matrix, determinant_list, determinant_list_cons_rows,
    determinant_list_nil_rows, list_index_or_zero, list_index_or_zero_cons_self,
    list_index_or_zero_cons_other, matrix_det_list, matrix_det_list_cons_rows,
    matrix_entry, matrix_entry_one_diag, matrix_entry_one_off_diag, matrix_entry_transpose,
    matrix_entry_zero, matrix_entry_mul, matrix_mul, matrix_mul_term, matrix_one,
    matrix_swap_rows, matrix_entry_swap_rows, matrix_swap_rows_entry, matrix_transpose,
    matrix_zero, square_matrix_entry_function
from data.fin.fin_sum import fin_sum
from data.fin.fin_sum_enum import fin_sum_suc_eq_sum_fin_enum_suc
from algebra.add_comm_monoid import AddCommMonoid
from algebra.add_semigroup import add_fn
from algebra.add_group import inverse_inverse
from comm_ring import CommRing
from list import List, map, sum, map_singleton, remove_one_cons_eq, remove_one_cons_neq,
    sum_map_of_pointwise, sum_map_remove_one, map_sum_add, unique_implies_tail_unique,
    remove_one_unique, remove_one_unique_not_contains_self, remove_one_contains_other
from algebra.ring.ring import Ring, alternating_sign, alternating_sign_zero,
    alternating_sign_suc, alternating_sign_suc_mul, mul_neg_left, mul_neg_right, mul_neg_one_left,
    mul_neg_one_right

numerals Nat

/// The canonical determinant of a positive-dimensional square matrix, using the
/// successor finite-index enumeration for both rows and columns.
define matrix_det_suc[R: Ring](n: Nat, a: Matrix[R, n.suc, n.suc]) -> R {
    matrix_det_list[R](n.suc, a, fin_enum_suc(n), fin_enum_suc(n))
}

/// The positive-dimensional determinant unfolds to the existing determinant-list
/// expansion over the canonical successor enumeration.
theorem matrix_det_suc_unfold[R: Ring](n: Nat, a: Matrix[R, n.suc, n.suc]) {
    matrix_det_suc[R](n, a) = matrix_det_list[R](n.suc, a, fin_enum_suc(n), fin_enum_suc(n))
}

/// The row/column list used by `matrix_det_suc` has exactly the ambient size.
theorem matrix_det_suc_enum_length(n: Nat) {
    fin_enum_suc(n).length = n.suc
} by {
    fin_enum_suc_length(n)
}

/// The row/column list used by `matrix_det_suc` contains every finite index.
theorem matrix_det_suc_enum_contains(n: Nat, i: Fin[n.suc]) {
    fin_enum_suc(n).contains(i)
} by {
    fin_enum_suc_contains(n, i)
}

/// The row/column list used by `matrix_det_suc` has no duplicate indices.
theorem matrix_det_suc_enum_is_unique(n: Nat) {
    fin_enum_suc(n).is_unique
} by {
    fin_enum_suc_is_unique(n)
}

/// In dimension one, the successor enumerator is the singleton first index.
theorem fin_enum_suc_zero_eq_singleton_zero {
    fin_enum_suc(Nat.0) = List.singleton(fin_zero(Nat.0))
} by {
    let z = Nat.0
    z < z.suc
    fin_of_nat_suc_new_of_lt(z, z)
    fin_zero_new(z)
    Option.some(fin_of_nat_suc(z, z)) = Option.some(fin_zero(z))
    some_injective[Fin[z.suc]](fin_of_nat_suc(z, z), fin_zero(z))
    fin_of_nat_suc(z, z) = fin_zero(z)
    z.range = List.nil[Nat]
    z.range.append(z) = List.singleton(z)
    map_singleton[Nat, Fin[z.suc]](fin_of_nat_suc(z), z)
}

/// The sum of a singleton list is its only term.
theorem sum_singleton_ring[R: Ring](x: R) {
    sum[R](List.singleton(x)) = x
} by {
    sum[R](List.cons(x, List.nil[R])) = x + sum[R](List.nil[R])
    sum[R](List.nil[R]) = R.0
    x + R.0 = x
}

/// Mapping a pointwise-zero function over a list has zero sum.
theorem sum_map_zero_ring[T, R: Ring](items: List[T], f: T -> R) {
    (forall(item: T) { f(item) = R.0 }) implies sum[R](map[T, R](items, f)) = R.0
} by {
    if forall(item: T) { f(item) = R.0 } {
        define p(xs: List[T]) -> Bool {
            sum[R](map[T, R](xs, f)) = R.0
        }

        map[T, R](List.nil[T], f) = List.nil[R]
        sum[R](List.nil[R]) = R.0
        p(List.nil[T])

        forall(head: T, tail: List[T]) {
            if p(tail) {
                f(head) = R.0
                map[T, R](List.cons(head, tail), f) = List.cons(f(head), map[T, R](tail, f))
                sum[R](map[T, R](tail, f)) = R.0
                sum[R](List.cons(f(head), map[T, R](tail, f))) = f(head) + sum[R](map[T, R](tail, f))
                f(head) + sum[R](map[T, R](tail, f)) = R.0
                sum[R](map[T, R](List.cons(head, tail), f)) = R.0
                p(List.cons(head, tail))
            }
            p(tail) implies p(List.cons(head, tail))
        }

        p(List.nil[T]) and forall(head: T, tail: List[T]) { p(tail) implies p(List.cons(head, tail)) }
        List.induction(p)
        forall(xs: List[T]) { p(xs) }
        p(items)
        sum[R](map[T, R](items, f)) = R.0
    }
}

/// If the first expanded row of a matrix determinant-list is zero, the determinant-list is zero.
theorem matrix_det_list_zero_cons_rows[R: Ring](n: Nat, row: Fin[n], tail_rows: List[Fin[n]], cols: List[Fin[n]]) {
    matrix_det_list[R](n, matrix_zero[R](n, n), List.cons(row, tail_rows), cols) = R.0
} by {
    matrix_det_list_cons_rows[R](n, matrix_zero[R](n, n), row, tail_rows, cols)
    forall(col: Fin[n]) {
        square_matrix_entry_function[R](n, matrix_zero[R](n, n), row, col) =
            matrix_entry[R](n, n, matrix_zero[R](n, n), row, col)
        matrix_entry_zero[R](n, n, row, col)
        square_matrix_entry_function[R](n, matrix_zero[R](n, n), row, col) = R.0
        R.0 * alternating_sign[R](list_index_or_zero[Fin[n]](cols, col)) = R.0
        square_matrix_entry_function[R](n, matrix_zero[R](n, n), row, col) *
            alternating_sign[R](list_index_or_zero[Fin[n]](cols, col)) = R.0
        R.0 * determinant_list[R, Fin[n]](square_matrix_entry_function[R](n, matrix_zero[R](n, n)), tail_rows, cols.remove_one(col)) = R.0
        square_matrix_entry_function[R](n, matrix_zero[R](n, n), row, col) *
            alternating_sign[R](list_index_or_zero[Fin[n]](cols, col)) *
            determinant_list[R, Fin[n]](square_matrix_entry_function[R](n, matrix_zero[R](n, n)), tail_rows, cols.remove_one(col)) = R.0
    }
    matrix_det_list[R](n, matrix_zero[R](n, n), List.cons(row, tail_rows), cols) =
        sum[R](map[Fin[n], R](cols, function(col: Fin[n]) {
            square_matrix_entry_function[R](n, matrix_zero[R](n, n), row, col) *
                alternating_sign[R](list_index_or_zero[Fin[n]](cols, col)) *
                determinant_list[R, Fin[n]](square_matrix_entry_function[R](n, matrix_zero[R](n, n)), tail_rows, cols.remove_one(col))
        }))
    sum_map_zero_ring[Fin[n], R](cols, function(col: Fin[n]) {
        square_matrix_entry_function[R](n, matrix_zero[R](n, n), row, col) *
            alternating_sign[R](list_index_or_zero[Fin[n]](cols, col)) *
            determinant_list[R, Fin[n]](square_matrix_entry_function[R](n, matrix_zero[R](n, n)), tail_rows, cols.remove_one(col))
    })
    sum[R](map[Fin[n], R](cols, function(col: Fin[n]) {
        square_matrix_entry_function[R](n, matrix_zero[R](n, n), row, col) *
            alternating_sign[R](list_index_or_zero[Fin[n]](cols, col)) *
            determinant_list[R, Fin[n]](square_matrix_entry_function[R](n, matrix_zero[R](n, n)), tail_rows, cols.remove_one(col))
    })) = R.0
    matrix_det_list[R](n, matrix_zero[R](n, n), List.cons(row, tail_rows), cols) = R.0
}

/// The determinant-list expansion over matching singleton row and column lists
/// is the single diagonal entry.
theorem determinant_list_singleton[R: Ring, I](entry: (I, I) -> R, item: I) {
    determinant_list[R, I](entry, List.singleton(item), List.singleton(item)) = entry(item, item)
} by {
    let cols = List.singleton(item)
    determinant_list_cons_rows[R, I](entry, item, List.nil[I], cols)
    list_index_or_zero_cons_self[I](item, List.nil[I])
    list_index_or_zero[I](cols, item) = Nat.0
    alternating_sign_zero[R]
    remove_one_cons_eq[I](item, List.nil[I])
    cols.remove_one(item) = List.nil[I]
    determinant_list_nil_rows[R, I](entry, List.nil[I])
    map_singleton[I, R](function(col: I) {
        entry(item, col) *
            alternating_sign[R](list_index_or_zero[I](cols, col)) *
            determinant_list[R, I](entry, List.nil[I], cols.remove_one(col))
    }, item)
    sum_singleton_ring[R](entry(item, item) * R.1 * R.1)
    entry(item, item) * R.1 = entry(item, item)
    entry(item, item) * R.1 * R.1 = entry(item, item)
    determinant_list[R, I](entry, cols, cols) = entry(item, item)
}

/// Matrix determinant-list over matching singleton row and column lists is the single diagonal entry.
theorem matrix_det_list_singleton[R: Ring](n: Nat, a: Matrix[R, n, n], item: Fin[n]) {
    matrix_det_list[R](n, a, List.singleton(item), List.singleton(item)) = matrix_entry[R](n, n, a, item, item)
} by {
    determinant_list_singleton[R, Fin[n]](square_matrix_entry_function[R](n, a), item)
    square_matrix_entry_function[R](n, a, item, item) = matrix_entry[R](n, n, a, item, item)
}

/// Transposing a matrix does not change the determinant-list over matching singleton lists.
theorem matrix_det_list_singleton_transpose[R: Ring](n: Nat, a: Matrix[R, n, n], item: Fin[n]) {
    matrix_det_list[R](n, matrix_transpose[R](n, n, a), List.singleton(item), List.singleton(item)) =
        matrix_det_list[R](n, a, List.singleton(item), List.singleton(item))
} by {
    matrix_det_list_singleton[R](n, matrix_transpose[R](n, n, a), item)
    matrix_det_list_singleton[R](n, a, item)
    matrix_entry_transpose[R](n, n, a, item, item)
}

/// The canonical determinant in dimension one is the determinant-list over the singleton row and column.
theorem matrix_det_suc_zero_eq_det_list_singleton[R: Ring](n: Nat, a: Matrix[R, n.suc, n.suc]) {
    n = Nat.0 implies matrix_det_suc[R](n, a) = matrix_det_list[R](n.suc, a, List.singleton(fin_zero(n)), List.singleton(fin_zero(n)))
} by {
    if n = Nat.0 {
        let first = fin_zero(n)
        let cols = List.singleton(first)
        fin_enum_suc_zero_eq_singleton_zero
        fin_enum_suc(n) = cols
        matrix_det_suc_unfold[R](n, a)
        matrix_det_suc[R](n, a) = matrix_det_list[R](n.suc, a, cols, cols)
    }
}

/// The canonical determinant of a `1 x 1` matrix is its only entry.
theorem matrix_det_suc_zero_eq_entry[R: Ring](n: Nat, a: Matrix[R, n.suc, n.suc]) {
    n = Nat.0 implies matrix_det_suc[R](n, a) = matrix_entry[R](n.suc, n.suc, a, fin_zero(n), fin_zero(n))
} by {
    if n = Nat.0 {
        matrix_det_suc_zero_eq_det_list_singleton[R](n, a)
        matrix_det_list_singleton[R](n.suc, a, fin_zero(n))
    }
}

/// The canonical determinant of a positive-dimensional zero matrix is zero.
theorem matrix_det_suc_zero[R: Ring](n: Nat) {
    matrix_det_suc[R](n, matrix_zero[R](n.suc, n.suc)) = R.0
} by {
    let rows = fin_enum_suc(n)
    fin_enum_suc_length(n)
    match rows {
        List.nil {
            rows.length = Nat.0
            rows.length = n.suc
            n.suc = Nat.0
            false
        }
        List.cons(row, tail_rows) {
            matrix_det_suc_unfold[R](n, matrix_zero[R](n.suc, n.suc))
            matrix_det_list_zero_cons_rows[R](n.suc, row, tail_rows, rows)
            matrix_det_suc[R](n, matrix_zero[R](n.suc, n.suc)) = R.0
        }
    }
}

/// The canonical determinant of the `1 x 1` identity matrix is one.
theorem matrix_det_suc_one_by_one_identity[R: Ring] {
    matrix_det_suc[R](Nat.0, matrix_one[R](Nat.0.suc)) = R.1
} by {
    let n = Nat.0
    let first = fin_zero(n)
    matrix_det_suc_zero_eq_entry[R](n, matrix_one[R](n.suc))
    matrix_entry_one_diag[R](n.suc, first)
    matrix_det_suc[R](n, matrix_one[R](n.suc)) =
        matrix_entry[R](n.suc, n.suc, matrix_one[R](n.suc), first, first)
    matrix_det_suc[R](n, matrix_one[R](n.suc)) = R.1
}

/// The canonical enumeration of `Fin[2]` is the two successor indices in order.
theorem fin_enum_suc_one_eq_two {
    fin_enum_suc(Nat.1) = List.cons(fin_zero(Nat.1), List.cons(fin_last(Nat.1), List.nil[Fin[Nat.1.suc]]))
} by {
    let n = Nat.1
    let z = Nat.0
    z < z.suc
    n.suc.range = n.range.append(n)
    n.range = z.range.append(z)
    z.range = List.nil[Nat]
    z.range.append(z) = List.singleton(z)
    n.range = List.singleton(z)
    n.suc.range = List.singleton(z).append(n)
    List.singleton(z).append(n) = List.singleton(z) + List.singleton(n)
    List.singleton(z) + List.singleton(n) = List.cons(z, List.nil[Nat]) + List.singleton(n)
    List.cons(z, List.nil[Nat]) + List.singleton(n) = List.cons(z, List.nil[Nat] + List.singleton(n))
    List.nil[Nat] + List.singleton(n) = List.singleton(n)
    List.cons(z, List.nil[Nat] + List.singleton(n)) = List.cons(z, List.singleton(n))
    List.singleton(z) + List.singleton(n) = List.cons(z, List.singleton(n))
    List.singleton(z).append(n) = List.cons(z, List.singleton(n))
    n.suc.range = List.cons(z, List.singleton(n))
    fin_enum_suc(n) = map[Nat, Fin[n.suc]](n.suc.range, fin_of_nat_suc(n))
    fin_enum_suc(n) = map[Nat, Fin[n.suc]](List.cons(z, List.singleton(n)), fin_of_nat_suc(n))
    map[Nat, Fin[n.suc]](List.cons(z, List.singleton(n)), fin_of_nat_suc(n)) =
        List.cons(fin_of_nat_suc(n, z), map[Nat, Fin[n.suc]](List.singleton(n), fin_of_nat_suc(n)))
    map_singleton[Nat, Fin[n.suc]](fin_of_nat_suc(n), n)
    map[Nat, Fin[n.suc]](List.singleton(n), fin_of_nat_suc(n)) = List.singleton(fin_of_nat_suc(n, n))
    map[Nat, Fin[n.suc]](List.cons(z, List.singleton(n)), fin_of_nat_suc(n)) =
        List.cons(fin_of_nat_suc(n, z), List.singleton(fin_of_nat_suc(n, n)))
    z < n.suc
    fin_of_nat_suc_new_of_lt(n, z)
    fin_zero_new(n)
    Option.some(fin_of_nat_suc(n, z)) = Option.some(fin_zero(n))
    some_injective[Fin[n.suc]](fin_of_nat_suc(n, z), fin_zero(n))
    fin_of_nat_suc(n, z) = fin_zero(n)
    n < n.suc
    fin_of_nat_suc_new_of_lt(n, n)
    fin_last_new(n)
    Option.some(fin_of_nat_suc(n, n)) = Option.some(fin_last(n))
    some_injective[Fin[n.suc]](fin_of_nat_suc(n, n), fin_last(n))
    fin_of_nat_suc(n, n) = fin_last(n)
    fin_enum_suc(n) = List.cons(fin_zero(n), List.singleton(fin_last(n)))
}

/// The determinant-list over a singleton row and a singleton column is the single entry.
///
/// This differs from `determinant_list_singleton` in that the row and the column
/// need not coincide.
theorem determinant_list_singleton_row_col[R: Ring, I](entry: (I, I) -> R, row: I, col: I) {
    determinant_list[R, I](entry, List.singleton(row), List.singleton(col)) = entry(row, col)
} by {
    let cols = List.singleton(col)
    determinant_list_cons_rows[R, I](entry, row, List.nil[I], cols)
    list_index_or_zero_cons_self[I](col, List.nil[I])
    list_index_or_zero[I](cols, col) = Nat.0
    alternating_sign_zero[R]
    remove_one_cons_eq[I](col, List.nil[I])
    cols.remove_one(col) = List.nil[I]
    determinant_list_nil_rows[R, I](entry, List.nil[I])
    map_singleton[I, R](function(col2: I) {
        entry(row, col2) *
            alternating_sign[R](list_index_or_zero[I](cols, col2)) *
            determinant_list[R, I](entry, List.nil[I], cols.remove_one(col2))
    }, col)
    sum_singleton_ring[R](entry(row, col) * R.1 * R.1)
    entry(row, col) * R.1 = entry(row, col)
    entry(row, col) * R.1 * R.1 = entry(row, col)
    determinant_list[R, I](entry, List.singleton(row), cols) = entry(row, col)
}

/// The two-element row list used in two-by-two expansions.
define two_row_list[I](r: I, r2: I) -> List[I] {
    List.cons(r, List.cons(r2, List.nil[I]))
}

/// The two-element column list used in two-by-two expansions.
define two_col_list[I](c: I, c2: I) -> List[I] {
    List.cons(c, List.cons(c2, List.nil[I]))
}

/// One summand of the two-by-two determinant expansion at its first row.
define det_two_two_term[R: Ring, I](entry: (I, I) -> R, r: I, r2: I, cols: List[I], col: I) -> R {
    entry(r, col) * alternating_sign[R](list_index_or_zero[I](cols, col)) *
        determinant_list[R, I](entry, List.cons(r2, List.nil[I]), cols.remove_one(col))
}

/// The determinant-list over two rows and two columns is the two-by-two difference of products.
theorem determinant_list_two_two[R: Ring, I](entry: (I, I) -> R, r: I, r2: I, c: I, c2: I) {
    c != c2 implies
        determinant_list[R, I](entry, two_row_list(r, r2), two_col_list(c, c2)) =
            entry(r, c) * entry(r2, c2) - entry(r, c2) * entry(r2, c)
} by {
    if c != c2 {
        determinant_list_cons_rows[R, I](entry, r, List.cons(r2, List.nil[I]), two_col_list(c, c2))
        determinant_list[R, I](entry, two_row_list(r, r2), two_col_list(c, c2)) =
            sum[R](map[I, R](two_col_list(c, c2), function(col: I) {
                entry(r, col) * alternating_sign[R](list_index_or_zero[I](two_col_list(c, c2), col)) *
                    determinant_list[R, I](entry, List.cons(r2, List.nil[I]), two_col_list(c, c2).remove_one(col))
            }))
        forall(col: I) {
            if two_col_list(c, c2).contains(col) {
                det_two_two_term[R, I](entry, r, r2, two_col_list(c, c2), col) =
                    entry(r, col) * alternating_sign[R](list_index_or_zero[I](two_col_list(c, c2), col)) *
                        determinant_list[R, I](entry, List.cons(r2, List.nil[I]), two_col_list(c, c2).remove_one(col))
            }
        }
        sum_map_of_pointwise[I, R](two_col_list(c, c2), det_two_two_term[R, I](entry, r, r2, two_col_list(c, c2)),
            function(col: I) {
                entry(r, col) * alternating_sign[R](list_index_or_zero[I](two_col_list(c, c2), col)) *
                    determinant_list[R, I](entry, List.cons(r2, List.nil[I]), two_col_list(c, c2).remove_one(col))
            })
        sum[R](map[I, R](two_col_list(c, c2), det_two_two_term[R, I](entry, r, r2, two_col_list(c, c2)))) =
            sum[R](map[I, R](two_col_list(c, c2), function(col: I) {
                entry(r, col) * alternating_sign[R](list_index_or_zero[I](two_col_list(c, c2), col)) *
                    determinant_list[R, I](entry, List.cons(r2, List.nil[I]), two_col_list(c, c2).remove_one(col))
            }))
        determinant_list[R, I](entry, two_row_list(r, r2), two_col_list(c, c2)) =
            sum[R](map[I, R](two_col_list(c, c2), det_two_two_term[R, I](entry, r, r2, two_col_list(c, c2))))
        two_col_list(c, c2).contains(c)
        sum_map_remove_one[I, R](two_col_list(c, c2), c, det_two_two_term[R, I](entry, r, r2, two_col_list(c, c2)))
        det_two_two_term[R, I](entry, r, r2, two_col_list(c, c2), c) +
            sum[R](map[I, R](two_col_list(c, c2).remove_one(c), det_two_two_term[R, I](entry, r, r2, two_col_list(c, c2)))) =
            sum[R](map[I, R](two_col_list(c, c2), det_two_two_term[R, I](entry, r, r2, two_col_list(c, c2))))
        remove_one_cons_eq(c, List.cons(c2, List.nil[I]))
        two_col_list(c, c2).remove_one(c) = List.cons(c2, List.nil[I])
        sum[R](map[I, R](List.cons(c2, List.nil[I]), det_two_two_term[R, I](entry, r, r2, two_col_list(c, c2)))) =
            det_two_two_term[R, I](entry, r, r2, two_col_list(c, c2), c2) +
                sum[R](map[I, R](List.nil[I], det_two_two_term[R, I](entry, r, r2, two_col_list(c, c2))))
        sum[R](map[I, R](List.nil[I], det_two_two_term[R, I](entry, r, r2, two_col_list(c, c2)))) = R.0
        sum[R](map[I, R](two_col_list(c, c2).remove_one(c), det_two_two_term[R, I](entry, r, r2, two_col_list(c, c2)))) =
            det_two_two_term[R, I](entry, r, r2, two_col_list(c, c2), c2) + R.0
        sum[R](map[I, R](two_col_list(c, c2), det_two_two_term[R, I](entry, r, r2, two_col_list(c, c2)))) =
            det_two_two_term[R, I](entry, r, r2, two_col_list(c, c2), c) +
                sum[R](map[I, R](two_col_list(c, c2).remove_one(c), det_two_two_term[R, I](entry, r, r2, two_col_list(c, c2))))
        sum[R](map[I, R](two_col_list(c, c2), det_two_two_term[R, I](entry, r, r2, two_col_list(c, c2)))) =
            det_two_two_term[R, I](entry, r, r2, two_col_list(c, c2), c) +
                (det_two_two_term[R, I](entry, r, r2, two_col_list(c, c2), c2) + R.0)
        determinant_list[R, I](entry, two_row_list(r, r2), two_col_list(c, c2)) =
            det_two_two_term[R, I](entry, r, r2, two_col_list(c, c2), c) +
                (det_two_two_term[R, I](entry, r, r2, two_col_list(c, c2), c2) + R.0)
        det_two_two_term[R, I](entry, r, r2, two_col_list(c, c2), c) +
            (det_two_two_term[R, I](entry, r, r2, two_col_list(c, c2), c2) + R.0) =
            det_two_two_term[R, I](entry, r, r2, two_col_list(c, c2), c) +
                det_two_two_term[R, I](entry, r, r2, two_col_list(c, c2), c2)
        determinant_list[R, I](entry, two_row_list(r, r2), two_col_list(c, c2)) =
            det_two_two_term[R, I](entry, r, r2, two_col_list(c, c2), c) +
                det_two_two_term[R, I](entry, r, r2, two_col_list(c, c2), c2)

        list_index_or_zero_cons_self(c, List.cons(c2, List.nil[I]))
        list_index_or_zero[I](two_col_list(c, c2), c) = Nat.0
        alternating_sign_zero[R]
        det_two_two_term[R, I](entry, r, r2, two_col_list(c, c2), c) =
            entry(r, c) * alternating_sign[R](Nat.0) *
                determinant_list[R, I](entry, List.cons(r2, List.nil[I]), two_col_list(c, c2).remove_one(c))
        alternating_sign[R](Nat.0) = R.1
        det_two_two_term[R, I](entry, r, r2, two_col_list(c, c2), c) = entry(r, c) * R.1 *
            determinant_list[R, I](entry, List.cons(r2, List.nil[I]), List.cons(c2, List.nil[I]))
        determinant_list_singleton_row_col(entry, r2, c2)
        determinant_list[R, I](entry, List.cons(r2, List.nil[I]), List.cons(c2, List.nil[I])) = entry(r2, c2)
        det_two_two_term[R, I](entry, r, r2, two_col_list(c, c2), c) = entry(r, c) * R.1 * entry(r2, c2)
        entry(r, c) * R.1 = entry(r, c)
        det_two_two_term[R, I](entry, r, r2, two_col_list(c, c2), c) = entry(r, c) * entry(r2, c2)

        c != c2
        list_index_or_zero_cons_other(c, List.cons(c2, List.nil[I]), c2)
        list_index_or_zero[I](two_col_list(c, c2), c2) =
            list_index_or_zero[I](List.cons(c2, List.nil[I]), c2).suc
        list_index_or_zero_cons_self(c2, List.nil[I])
        list_index_or_zero[I](List.cons(c2, List.nil[I]), c2) = Nat.0
        list_index_or_zero[I](two_col_list(c, c2), c2) = Nat.0.suc
        alternating_sign_suc[R](Nat.0)
        alternating_sign[R](Nat.0.suc) = -alternating_sign[R](Nat.0)
        alternating_sign[R](Nat.0) = R.1
        alternating_sign[R](Nat.0.suc) = -R.1
        alternating_sign_suc_mul[R](Nat.0)
        alternating_sign[R](Nat.0.suc) = -R.1 * alternating_sign[R](Nat.0)
        alternating_sign[R](Nat.0.suc) = -R.1 * R.1
        remove_one_cons_neq(c, List.cons(c2, List.nil[I]), c2)
        List.cons(c, List.cons(c2, List.nil[I])).remove_one(c2) =
            List.cons(c, List.cons(c2, List.nil[I]).remove_one(c2))
        remove_one_cons_eq(c2, List.nil[I])
        List.cons(c2, List.nil[I]).remove_one(c2) = List.nil[I]
        two_col_list(c, c2).remove_one(c2) = List.cons(c, List.nil[I])
        det_two_two_term[R, I](entry, r, r2, two_col_list(c, c2), c2) =
            entry(r, c2) * alternating_sign[R](list_index_or_zero[I](two_col_list(c, c2), c2)) *
                determinant_list[R, I](entry, List.cons(r2, List.nil[I]), List.cons(c, List.nil[I]))
        det_two_two_term[R, I](entry, r, r2, two_col_list(c, c2), c2) = entry(r, c2) * (-R.1) *
            determinant_list[R, I](entry, List.cons(r2, List.nil[I]), List.cons(c, List.nil[I]))
        determinant_list_singleton_row_col(entry, r2, c)
        determinant_list[R, I](entry, List.cons(r2, List.nil[I]), List.cons(c, List.nil[I])) = entry(r2, c)
        det_two_two_term[R, I](entry, r, r2, two_col_list(c, c2), c2) = entry(r, c2) * (-R.1) * entry(r2, c)
        mul_neg_one_right[R](entry(r, c2))
        entry(r, c2) * (-R.1) = -entry(r, c2)
        det_two_two_term[R, I](entry, r, r2, two_col_list(c, c2), c2) = (-entry(r, c2)) * entry(r2, c)
        mul_neg_left[R](entry(r, c2), entry(r2, c))
        (-entry(r, c2)) * entry(r2, c) = -(entry(r, c2) * entry(r2, c))
        det_two_two_term[R, I](entry, r, r2, two_col_list(c, c2), c2) = -(entry(r, c2) * entry(r2, c))

        det_two_two_term[R, I](entry, r, r2, two_col_list(c, c2), c) +
            det_two_two_term[R, I](entry, r, r2, two_col_list(c, c2), c2) =
            entry(r, c) * entry(r2, c2) + (-(entry(r, c2) * entry(r2, c)))
        determinant_list[R, I](entry, two_row_list(r, r2), two_col_list(c, c2)) =
            entry(r, c) * entry(r2, c2) + (-(entry(r, c2) * entry(r2, c)))
        entry(r, c) * entry(r2, c2) + (-(entry(r, c2) * entry(r2, c))) =
            entry(r, c) * entry(r2, c2) - entry(r, c2) * entry(r2, c)
        determinant_list[R, I](entry, two_row_list(r, r2), two_col_list(c, c2)) =
            entry(r, c) * entry(r2, c2) - entry(r, c2) * entry(r2, c)
    }
}

/// The canonical determinant of a `2 x 2` matrix is the difference of diagonal
/// and off-diagonal products.
theorem matrix_det_suc_two_by_two[R: Ring](a: Matrix[R, Nat.1.suc, Nat.1.suc]) {
    matrix_det_suc[R](Nat.1, a) =
        matrix_entry[R](Nat.1.suc, Nat.1.suc, a, fin_zero(Nat.1), fin_zero(Nat.1)) *
            matrix_entry[R](Nat.1.suc, Nat.1.suc, a, fin_last(Nat.1), fin_last(Nat.1)) -
        matrix_entry[R](Nat.1.suc, Nat.1.suc, a, fin_zero(Nat.1), fin_last(Nat.1)) *
            matrix_entry[R](Nat.1.suc, Nat.1.suc, a, fin_last(Nat.1), fin_zero(Nat.1))
} by {
    matrix_det_suc_unfold[R](Nat.1, a)
    matrix_det_suc[R](Nat.1, a) =
        matrix_det_list[R](Nat.1.suc, a, fin_enum_suc(Nat.1), fin_enum_suc(Nat.1))
    fin_enum_suc_one_eq_two
    fin_enum_suc(Nat.1) =
        List.cons(fin_zero(Nat.1), List.cons(fin_last(Nat.1), List.nil[Fin[Nat.1.suc]]))
    matrix_det_list[R](Nat.1.suc, a, fin_enum_suc(Nat.1), fin_enum_suc(Nat.1)) =
        matrix_det_list[R](Nat.1.suc, a, List.cons(fin_zero(Nat.1), List.cons(fin_last(Nat.1), List.nil[Fin[Nat.1.suc]])),
            List.cons(fin_zero(Nat.1), List.cons(fin_last(Nat.1), List.nil[Fin[Nat.1.suc]])))
    matrix_det_list[R](Nat.1.suc, a, two_row_list[Fin[Nat.1.suc]](fin_zero(Nat.1), fin_last(Nat.1)),
        two_col_list[Fin[Nat.1.suc]](fin_zero(Nat.1), fin_last(Nat.1))) =
        determinant_list[R, Fin[Nat.1.suc]](square_matrix_entry_function[R](Nat.1.suc, a),
            two_row_list[Fin[Nat.1.suc]](fin_zero(Nat.1), fin_last(Nat.1)),
            two_col_list[Fin[Nat.1.suc]](fin_zero(Nat.1), fin_last(Nat.1)))
    fin_zero_value(Nat.1)
    fin_zero(Nat.1).value = Nat.0
    fin_last_value(Nat.1)
    fin_last(Nat.1).value = Nat.1
    Nat.0 != Nat.1
    if fin_zero(Nat.1) = fin_last(Nat.1) {
        fin_zero(Nat.1).value = fin_last(Nat.1).value
        Nat.0 = Nat.1
        false
    }
    not fin_zero(Nat.1) = fin_last(Nat.1)
    fin_zero(Nat.1) != fin_last(Nat.1)
    determinant_list_two_two[R, Fin[Nat.1.suc]](square_matrix_entry_function[R](Nat.1.suc, a),
        fin_zero(Nat.1), fin_last(Nat.1), fin_zero(Nat.1), fin_last(Nat.1))
    determinant_list[R, Fin[Nat.1.suc]](square_matrix_entry_function[R](Nat.1.suc, a),
        two_row_list[Fin[Nat.1.suc]](fin_zero(Nat.1), fin_last(Nat.1)),
        two_col_list[Fin[Nat.1.suc]](fin_zero(Nat.1), fin_last(Nat.1))) =
        square_matrix_entry_function[R](Nat.1.suc, a, fin_zero(Nat.1), fin_zero(Nat.1)) *
            square_matrix_entry_function[R](Nat.1.suc, a, fin_last(Nat.1), fin_last(Nat.1)) -
        square_matrix_entry_function[R](Nat.1.suc, a, fin_zero(Nat.1), fin_last(Nat.1)) *
            square_matrix_entry_function[R](Nat.1.suc, a, fin_last(Nat.1), fin_zero(Nat.1))
    square_matrix_entry_function[R](Nat.1.suc, a, fin_zero(Nat.1), fin_zero(Nat.1)) =
        matrix_entry[R](Nat.1.suc, Nat.1.suc, a, fin_zero(Nat.1), fin_zero(Nat.1))
    square_matrix_entry_function[R](Nat.1.suc, a, fin_last(Nat.1), fin_last(Nat.1)) =
        matrix_entry[R](Nat.1.suc, Nat.1.suc, a, fin_last(Nat.1), fin_last(Nat.1))
    square_matrix_entry_function[R](Nat.1.suc, a, fin_zero(Nat.1), fin_last(Nat.1)) =
        matrix_entry[R](Nat.1.suc, Nat.1.suc, a, fin_zero(Nat.1), fin_last(Nat.1))
    square_matrix_entry_function[R](Nat.1.suc, a, fin_last(Nat.1), fin_zero(Nat.1)) =
        matrix_entry[R](Nat.1.suc, Nat.1.suc, a, fin_last(Nat.1), fin_zero(Nat.1))
    determinant_list[R, Fin[Nat.1.suc]](square_matrix_entry_function[R](Nat.1.suc, a),
        two_row_list[Fin[Nat.1.suc]](fin_zero(Nat.1), fin_last(Nat.1)),
        two_col_list[Fin[Nat.1.suc]](fin_zero(Nat.1), fin_last(Nat.1))) =
        matrix_entry[R](Nat.1.suc, Nat.1.suc, a, fin_zero(Nat.1), fin_zero(Nat.1)) *
            matrix_entry[R](Nat.1.suc, Nat.1.suc, a, fin_last(Nat.1), fin_last(Nat.1)) -
        matrix_entry[R](Nat.1.suc, Nat.1.suc, a, fin_zero(Nat.1), fin_last(Nat.1)) *
            matrix_entry[R](Nat.1.suc, Nat.1.suc, a, fin_last(Nat.1), fin_zero(Nat.1))
    matrix_det_list[R](Nat.1.suc, a, two_row_list[Fin[Nat.1.suc]](fin_zero(Nat.1), fin_last(Nat.1)),
        two_col_list[Fin[Nat.1.suc]](fin_zero(Nat.1), fin_last(Nat.1))) =
        matrix_entry[R](Nat.1.suc, Nat.1.suc, a, fin_zero(Nat.1), fin_zero(Nat.1)) *
            matrix_entry[R](Nat.1.suc, Nat.1.suc, a, fin_last(Nat.1), fin_last(Nat.1)) -
        matrix_entry[R](Nat.1.suc, Nat.1.suc, a, fin_zero(Nat.1), fin_last(Nat.1)) *
            matrix_entry[R](Nat.1.suc, Nat.1.suc, a, fin_last(Nat.1), fin_zero(Nat.1))
    matrix_det_list[R](Nat.1.suc, a, List.cons(fin_zero(Nat.1), List.cons(fin_last(Nat.1), List.nil[Fin[Nat.1.suc]])),
        List.cons(fin_zero(Nat.1), List.cons(fin_last(Nat.1), List.nil[Fin[Nat.1.suc]]))) =
        matrix_entry[R](Nat.1.suc, Nat.1.suc, a, fin_zero(Nat.1), fin_zero(Nat.1)) *
            matrix_entry[R](Nat.1.suc, Nat.1.suc, a, fin_last(Nat.1), fin_last(Nat.1)) -
        matrix_entry[R](Nat.1.suc, Nat.1.suc, a, fin_zero(Nat.1), fin_last(Nat.1)) *
            matrix_entry[R](Nat.1.suc, Nat.1.suc, a, fin_last(Nat.1), fin_zero(Nat.1))
    matrix_det_suc[R](Nat.1, a) =
        matrix_entry[R](Nat.1.suc, Nat.1.suc, a, fin_zero(Nat.1), fin_zero(Nat.1)) *
            matrix_entry[R](Nat.1.suc, Nat.1.suc, a, fin_last(Nat.1), fin_last(Nat.1)) -
        matrix_entry[R](Nat.1.suc, Nat.1.suc, a, fin_zero(Nat.1), fin_last(Nat.1)) *
            matrix_entry[R](Nat.1.suc, Nat.1.suc, a, fin_last(Nat.1), fin_zero(Nat.1))
}


/// Negation of a reversed difference.
theorem sub_neg_rev[R: CommRing](a: R, b: R) {
    a - b = -(b - a)
} by {
}

/// A difference of an element with itself is zero.
theorem sub_self_zero[R: CommRing](x: R) {
    x - x = R.0
} by {
}

/// Regrouping an addition with a nested pair.
theorem add_regroup_mid[R: CommRing](a: R, b: R, c: R, d: R) {
    a + (b + (c + d)) = a + (c + (b + d))
} by {
}

/// Folding negated summands into subtraction.
theorem add_sub_fold4[R: CommRing](a: R, b: R, c: R, d: R) {
    a + (-b) + (-c) + d = a - b - c + d
} by {
}

/// Factoring a difference out of a product difference.
theorem sub_factor[R: CommRing](x: R, y: R, z: R) {
    x * y - x * z = x * (y - z)
} by {
}

/// Swapping two adjacent factors in a four-fold product.
theorem mul_comm4[R: CommRing](a: R, b: R, c: R, d: R) {
    a * b * c * d = a * c * b * d
} by {
}

/// Swapping two adjacent factors in a three-fold product.
theorem mul_comm3[R: CommRing](a: R, b: R, c: R) {
    a * b * c = a * c * b
} by {
}

/// Splitting a two-fold sum difference into paired differences.
theorem sub_split_pair[R: CommRing](a: R, b: R, c: R, d: R) {
    (a + b) - (c + d) = (a - c) + (b - d)
} by {
}

/// (x + y)(z + w) expands to the four products.
theorem expand_sum_prod[R: CommRing](x: R, y: R, z: R, w: R) {
    (x + y) * (z + w) = x * z + x * w + y * z + y * w
} by {
    (x + y) * (z + w) = (x + y) * z + (x + y) * w
    (x + y) * z = x * z + y * z
    (x + y) * w = x * w + y * w
    (x + y) * z + (x + y) * w = x * z + y * z + (x * w + y * w)
    x * z + y * z + (x * w + y * w) = x * z + (y * z + (x * w + y * w))
    add_regroup_mid(x * z, y * z, x * w, y * w)
    x * z + (y * z + (x * w + y * w)) = x * z + (x * w + (y * z + y * w))
    x * z + (x * w + (y * z + y * w)) = x * z + x * w + y * z + y * w
    (x + y) * (z + w) = x * z + x * w + y * z + y * w
}

/// (x - y)(z - w) expands to the four signed products.
theorem expand_diff_prod[R: CommRing](x: R, y: R, z: R, w: R) {
    (x - y) * (z - w) = x * z - x * w - y * z + y * w
} by {
    (x - y) * (z - w) = (x + (-y)) * (z + (-w))
    expand_sum_prod(x, -y, z, -w)
    (x + (-y)) * (z + (-w)) = x * z + x * (-w) + (-y) * z + (-y) * (-w)
    x * (-w) = -(x * w)
    (-y) * z = -(y * z)
    (-y) * (-w) = y * w
    x * z + x * (-w) + (-y) * z + (-y) * (-w) = x * z + (-(x * w)) + (-(y * z)) + y * w
    add_sub_fold4(x * z, x * w, y * z, y * w)
    x * z + (-(x * w)) + (-(y * z)) + y * w = x * z - x * w - y * z + y * w
    (x + (-y)) * (z + (-w)) = x * z - x * w - y * z + y * w
    (x - y) * (z - w) = x * z - x * w - y * z + y * w
}

/// Factoring a difference out of a right product difference.
theorem sub_factor_right[R: CommRing](x: R, y: R, w: R) {
    x * w - y * w = (x - y) * w
} by {
    x * w - y * w = w * x - w * y
    sub_factor(w, x, y)
    w * x - w * y = w * (x - y)
    w * (x - y) = (x - y) * w
    x * w - y * w = (x - y) * w
}

/// The two-by-two determinant product identity: det(AB) = det(A) det(B) at the entry level.
theorem det2_mul_identity[R: CommRing](a00: R, a01: R, a10: R, a11: R, b00: R, b01: R, b10: R, b11: R) {
    (a00 * b00 + a01 * b10) * (a10 * b01 + a11 * b11) -
        (a00 * b01 + a01 * b11) * (a10 * b00 + a11 * b10) =
        (a00 * a11 - a01 * a10) * (b00 * b11 - b01 * b10)
} by {
    expand_sum_prod(a00 * b00, a01 * b10, a10 * b01, a11 * b11)
    (a00 * b00 + a01 * b10) * (a10 * b01 + a11 * b11) =
        a00 * b00 * (a10 * b01) + a00 * b00 * (a11 * b11) + a01 * b10 * (a10 * b01) + a01 * b10 * (a11 * b11)
    expand_sum_prod(a00 * b01, a01 * b11, a10 * b00, a11 * b10)
    (a00 * b01 + a01 * b11) * (a10 * b00 + a11 * b10) =
        a00 * b01 * (a10 * b00) + a00 * b01 * (a11 * b10) + a01 * b11 * (a10 * b00) + a01 * b11 * (a11 * b10)
    (a00 * b00 + a01 * b10) * (a10 * b01 + a11 * b11) -
        (a00 * b01 + a01 * b11) * (a10 * b00 + a11 * b10) =
        (a00 * b00 * (a10 * b01) + a00 * b00 * (a11 * b11) + a01 * b10 * (a10 * b01) + a01 * b10 * (a11 * b11)) -
        (a00 * b01 * (a10 * b00) + a00 * b01 * (a11 * b10) + a01 * b11 * (a10 * b00) + a01 * b11 * (a11 * b10))
    (a00 * b00 * (a10 * b01) + a00 * b00 * (a11 * b11) + a01 * b10 * (a10 * b01) + a01 * b10 * (a11 * b11)) -
        (a00 * b01 * (a10 * b00) + a00 * b01 * (a11 * b10) + a01 * b11 * (a10 * b00) + a01 * b11 * (a11 * b10)) =
        ((a00 * b00 * (a10 * b01) + a00 * b00 * (a11 * b11)) + (a01 * b10 * (a10 * b01) + a01 * b10 * (a11 * b11))) -
        ((a00 * b01 * (a10 * b00) + a00 * b01 * (a11 * b10)) + (a01 * b11 * (a10 * b00) + a01 * b11 * (a11 * b10)))
    sub_split_pair(a00 * b00 * (a10 * b01) + a00 * b00 * (a11 * b11), a01 * b10 * (a10 * b01) + a01 * b10 * (a11 * b11),
        a00 * b01 * (a10 * b00) + a00 * b01 * (a11 * b10), a01 * b11 * (a10 * b00) + a01 * b11 * (a11 * b10))
    ((a00 * b00 * (a10 * b01) + a00 * b00 * (a11 * b11)) + (a01 * b10 * (a10 * b01) + a01 * b10 * (a11 * b11))) -
        ((a00 * b01 * (a10 * b00) + a00 * b01 * (a11 * b10)) + (a01 * b11 * (a10 * b00) + a01 * b11 * (a11 * b10))) =
        ((a00 * b00 * (a10 * b01) + a00 * b00 * (a11 * b11)) - (a00 * b01 * (a10 * b00) + a00 * b01 * (a11 * b10))) +
        ((a01 * b10 * (a10 * b01) + a01 * b10 * (a11 * b11)) - (a01 * b11 * (a10 * b00) + a01 * b11 * (a11 * b10)))
    sub_split_pair(a00 * b00 * (a10 * b01), a00 * b00 * (a11 * b11), a00 * b01 * (a10 * b00), a00 * b01 * (a11 * b10))
    (a00 * b00 * (a10 * b01) + a00 * b00 * (a11 * b11)) - (a00 * b01 * (a10 * b00) + a00 * b01 * (a11 * b10)) =
        (a00 * b00 * (a10 * b01) - a00 * b01 * (a10 * b00)) + (a00 * b00 * (a11 * b11) - a00 * b01 * (a11 * b10))
    sub_split_pair(a01 * b10 * (a10 * b01), a01 * b10 * (a11 * b11), a01 * b11 * (a10 * b00), a01 * b11 * (a11 * b10))
    (a01 * b10 * (a10 * b01) + a01 * b10 * (a11 * b11)) - (a01 * b11 * (a10 * b00) + a01 * b11 * (a11 * b10)) =
        (a01 * b10 * (a10 * b01) - a01 * b11 * (a10 * b00)) + (a01 * b10 * (a11 * b11) - a01 * b11 * (a11 * b10))
    (a00 * b00 + a01 * b10) * (a10 * b01 + a11 * b11) -
        (a00 * b01 + a01 * b11) * (a10 * b00 + a11 * b10) =
        (a00 * b00 * (a10 * b01) - a00 * b01 * (a10 * b00)) + (a00 * b00 * (a11 * b11) - a00 * b01 * (a11 * b10)) +
        (a01 * b10 * (a10 * b01) - a01 * b11 * (a10 * b00)) + (a01 * b10 * (a11 * b11) - a01 * b11 * (a11 * b10))
    mul_comm4(a00, b00, a10, b01)
    a00 * b00 * a10 * b01 = a00 * a10 * b00 * b01
    mul_comm4(a00, b01, a10, b00)
    a00 * b01 * a10 * b00 = a00 * a10 * b01 * b00
    mul_comm3(a00 * a10, b00, b01)
    a00 * a10 * b00 * b01 = a00 * a10 * b01 * b00
    a00 * b00 * a10 * b01 = a00 * b01 * a10 * b00
    a00 * b00 * (a10 * b01) = a00 * b01 * (a10 * b00)
    a00 * b00 * (a10 * b01) - a00 * b01 * (a10 * b00) =
        a00 * b00 * (a10 * b01) - a00 * b00 * (a10 * b01)
    a00 * b00 * (a10 * b01) - a00 * b00 * (a10 * b01) = R.0
    a00 * b00 * (a10 * b01) - a00 * b01 * (a10 * b00) = R.0
    mul_comm4(a00, b00, a11, b11)
    a00 * b00 * a11 * b11 = a00 * a11 * b00 * b11
    mul_comm4(a00, b01, a11, b10)
    a00 * b01 * a11 * b10 = a00 * a11 * b01 * b10
    sub_factor(a00 * a11, b00 * b11, b01 * b10)
    a00 * a11 * (b00 * b11) - a00 * a11 * (b01 * b10) = a00 * a11 * (b00 * b11 - b01 * b10)
    a00 * b00 * (a11 * b11) - a00 * b01 * (a11 * b10) = a00 * a11 * (b00 * b11 - b01 * b10)
    mul_comm4(a01, b10, a10, b01)
    a01 * b10 * a10 * b01 = a01 * a10 * b10 * b01
    mul_comm4(a01, b11, a10, b00)
    a01 * b11 * a10 * b00 = a01 * a10 * b11 * b00
    a01 * a10 * b10 * b01 = a01 * a10 * b01 * b10
    a01 * a10 * b11 * b00 = a01 * a10 * b00 * b11
    a01 * b10 * (a10 * b01) - a01 * b11 * (a10 * b00) = a01 * a10 * (b01 * b10) - a01 * a10 * (b00 * b11)
    sub_factor(a01 * a10, b01 * b10, b00 * b11)
    a01 * a10 * (b01 * b10) - a01 * a10 * (b00 * b11) = a01 * a10 * (b01 * b10 - b00 * b11)
    sub_neg_rev(b01 * b10, b00 * b11)
    b01 * b10 - b00 * b11 = -(b00 * b11 - b01 * b10)
    a01 * a10 * (b01 * b10 - b00 * b11) = a01 * a10 * (-(b00 * b11 - b01 * b10))
    a01 * a10 * (-(b00 * b11 - b01 * b10)) = -(a01 * a10 * (b00 * b11 - b01 * b10))
    a01 * a10 * (b01 * b10 - b00 * b11) = -(a01 * a10 * (b00 * b11 - b01 * b10))
    a01 * b10 * (a10 * b01) - a01 * b11 * (a10 * b00) = -(a01 * a10 * (b00 * b11 - b01 * b10))
    mul_comm4(a01, b10, a11, b11)
    a01 * b10 * a11 * b11 = a01 * a11 * b10 * b11
    mul_comm4(a01, b11, a11, b10)
    a01 * b11 * a11 * b10 = a01 * a11 * b11 * b10
    mul_comm3(a01 * a11, b10, b11)
    a01 * a11 * b10 * b11 = a01 * a11 * b11 * b10
    a01 * b10 * (a11 * b11) = a01 * b11 * (a11 * b10)
    a01 * b10 * (a11 * b11) - a01 * b11 * (a11 * b10) =
        a01 * b10 * (a11 * b11) - a01 * b10 * (a11 * b11)
    sub_self_zero(a01 * b10 * (a11 * b11))
    a01 * b10 * (a11 * b11) - a01 * b10 * (a11 * b11) = R.0
    a01 * b10 * (a11 * b11) - a01 * b11 * (a11 * b10) = R.0
    (a00 * b00 * (a10 * b01) - a00 * b01 * (a10 * b00)) + (a00 * b00 * (a11 * b11) - a00 * b01 * (a11 * b10)) +
        (a01 * b10 * (a10 * b01) - a01 * b11 * (a10 * b00)) + (a01 * b10 * (a11 * b11) - a01 * b11 * (a11 * b10)) =
        R.0 + (a00 * a11 * (b00 * b11 - b01 * b10)) + (-(a01 * a10 * (b00 * b11 - b01 * b10))) + R.0
    R.0 + (a00 * a11 * (b00 * b11 - b01 * b10)) + (-(a01 * a10 * (b00 * b11 - b01 * b10))) + R.0 =
        a00 * a11 * (b00 * b11 - b01 * b10) - a01 * a10 * (b00 * b11 - b01 * b10)
    sub_factor_right(a00 * a11, a01 * a10, b00 * b11 - b01 * b10)
    a00 * a11 * (b00 * b11 - b01 * b10) - a01 * a10 * (b00 * b11 - b01 * b10) =
        (a00 * a11 - a01 * a10) * (b00 * b11 - b01 * b10)
    (a00 * b00 + a01 * b10) * (a10 * b01 + a11 * b11) -
        (a00 * b01 + a01 * b11) * (a10 * b00 + a11 * b10) =
        (a00 * a11 - a01 * a10) * (b00 * b11 - b01 * b10)
}

/// The finite sum over `Fin[2]` splits into the two successor indices.
theorem fin_sum_two[R: AddCommMonoid](f: Fin[Nat.1.suc] -> R) {
    fin_sum[R](Nat.1.suc, f) = f(fin_zero(Nat.1)) + f(fin_last(Nat.1))
} by {
    fin_sum_suc_eq_sum_fin_enum_suc[R](Nat.1, f)
    fin_sum[R](Nat.1.suc, f) = sum[R](map[Fin[Nat.1.suc], R](fin_enum_suc(Nat.1), f))
    fin_enum_suc_one_eq_two
    fin_enum_suc(Nat.1) = List.cons(fin_zero(Nat.1), List.cons(fin_last(Nat.1), List.nil[Fin[Nat.1.suc]]))
    sum[R](map[Fin[Nat.1.suc], R](fin_enum_suc(Nat.1), f)) =
        sum[R](map[Fin[Nat.1.suc], R](List.cons(fin_zero(Nat.1), List.cons(fin_last(Nat.1), List.nil[Fin[Nat.1.suc]])), f))
    sum[R](map[Fin[Nat.1.suc], R](List.cons(fin_zero(Nat.1), List.cons(fin_last(Nat.1), List.nil[Fin[Nat.1.suc]])), f)) =
        f(fin_zero(Nat.1)) + sum[R](map[Fin[Nat.1.suc], R](List.cons(fin_last(Nat.1), List.nil[Fin[Nat.1.suc]]), f))
    sum[R](map[Fin[Nat.1.suc], R](List.cons(fin_last(Nat.1), List.nil[Fin[Nat.1.suc]]), f)) =
        f(fin_last(Nat.1)) + sum[R](map[Fin[Nat.1.suc], R](List.nil[Fin[Nat.1.suc]], f))
    sum[R](map[Fin[Nat.1.suc], R](List.nil[Fin[Nat.1.suc]], f)) = R.0
    f(fin_zero(Nat.1)) + (f(fin_last(Nat.1)) + R.0) =
        f(fin_zero(Nat.1)) + f(fin_last(Nat.1))
    sum[R](map[Fin[Nat.1.suc], R](List.cons(fin_zero(Nat.1), List.cons(fin_last(Nat.1), List.nil[Fin[Nat.1.suc]])), f)) =
        f(fin_zero(Nat.1)) + f(fin_last(Nat.1))
    sum[R](map[Fin[Nat.1.suc], R](fin_enum_suc(Nat.1), f)) = f(fin_zero(Nat.1)) + f(fin_last(Nat.1))
    fin_sum[R](Nat.1.suc, f) = f(fin_zero(Nat.1)) + f(fin_last(Nat.1))
}


/// The entries of a two-by-two matrix product are the two-term dot products.
theorem matrix_entry_mul_two_by_two[R: CommRing](a: Matrix[R, Nat.1.suc, Nat.1.suc], b: Matrix[R, Nat.1.suc, Nat.1.suc], i: Fin[Nat.1.suc], j: Fin[Nat.1.suc]) {
    matrix_entry[R](Nat.1.suc, Nat.1.suc, matrix_mul[R](Nat.1.suc, Nat.1.suc, Nat.1.suc, a, b), i, j) =
        matrix_entry[R](Nat.1.suc, Nat.1.suc, a, i, fin_zero(Nat.1)) *
            matrix_entry[R](Nat.1.suc, Nat.1.suc, b, fin_zero(Nat.1), j) +
        matrix_entry[R](Nat.1.suc, Nat.1.suc, a, i, fin_last(Nat.1)) *
            matrix_entry[R](Nat.1.suc, Nat.1.suc, b, fin_last(Nat.1), j)
} by {
    matrix_entry_mul[R](Nat.1.suc, Nat.1.suc, Nat.1.suc, a, b, i, j)
    matrix_entry[R](Nat.1.suc, Nat.1.suc, matrix_mul[R](Nat.1.suc, Nat.1.suc, Nat.1.suc, a, b), i, j) =
        fin_sum[R](Nat.1.suc, matrix_mul_term[R](Nat.1.suc, Nat.1.suc, Nat.1.suc, a, b, i, j))
    fin_sum_two[R](matrix_mul_term[R](Nat.1.suc, Nat.1.suc, Nat.1.suc, a, b, i, j))
    fin_sum[R](Nat.1.suc, matrix_mul_term[R](Nat.1.suc, Nat.1.suc, Nat.1.suc, a, b, i, j)) =
        matrix_mul_term[R](Nat.1.suc, Nat.1.suc, Nat.1.suc, a, b, i, j, fin_zero(Nat.1)) +
            matrix_mul_term[R](Nat.1.suc, Nat.1.suc, Nat.1.suc, a, b, i, j, fin_last(Nat.1))
    matrix_mul_term[R](Nat.1.suc, Nat.1.suc, Nat.1.suc, a, b, i, j, fin_zero(Nat.1)) =
        matrix_entry[R](Nat.1.suc, Nat.1.suc, a, i, fin_zero(Nat.1)) *
            matrix_entry[R](Nat.1.suc, Nat.1.suc, b, fin_zero(Nat.1), j)
    matrix_mul_term[R](Nat.1.suc, Nat.1.suc, Nat.1.suc, a, b, i, j, fin_last(Nat.1)) =
        matrix_entry[R](Nat.1.suc, Nat.1.suc, a, i, fin_last(Nat.1)) *
            matrix_entry[R](Nat.1.suc, Nat.1.suc, b, fin_last(Nat.1), j)
    matrix_entry[R](Nat.1.suc, Nat.1.suc, matrix_mul[R](Nat.1.suc, Nat.1.suc, Nat.1.suc, a, b), i, j) =
        matrix_entry[R](Nat.1.suc, Nat.1.suc, a, i, fin_zero(Nat.1)) *
            matrix_entry[R](Nat.1.suc, Nat.1.suc, b, fin_zero(Nat.1), j) +
        matrix_entry[R](Nat.1.suc, Nat.1.suc, a, i, fin_last(Nat.1)) *
            matrix_entry[R](Nat.1.suc, Nat.1.suc, b, fin_last(Nat.1), j)
}

/// The determinant of a two-by-two product is the product of the determinants.
theorem matrix_det_suc_two_by_two_mul[R: CommRing](a: Matrix[R, Nat.1.suc, Nat.1.suc], b: Matrix[R, Nat.1.suc, Nat.1.suc]) {
    matrix_det_suc[R](Nat.1, matrix_mul[R](Nat.1.suc, Nat.1.suc, Nat.1.suc, a, b)) =
        matrix_det_suc[R](Nat.1, a) * matrix_det_suc[R](Nat.1, b)
} by {
    matrix_det_suc_two_by_two[R](matrix_mul[R](Nat.1.suc, Nat.1.suc, Nat.1.suc, a, b))
    matrix_det_suc[R](Nat.1, matrix_mul[R](Nat.1.suc, Nat.1.suc, Nat.1.suc, a, b)) =
        matrix_entry[R](Nat.1.suc, Nat.1.suc, matrix_mul[R](Nat.1.suc, Nat.1.suc, Nat.1.suc, a, b), fin_zero(Nat.1), fin_zero(Nat.1)) *
            matrix_entry[R](Nat.1.suc, Nat.1.suc, matrix_mul[R](Nat.1.suc, Nat.1.suc, Nat.1.suc, a, b), fin_last(Nat.1), fin_last(Nat.1)) -
        matrix_entry[R](Nat.1.suc, Nat.1.suc, matrix_mul[R](Nat.1.suc, Nat.1.suc, Nat.1.suc, a, b), fin_zero(Nat.1), fin_last(Nat.1)) *
            matrix_entry[R](Nat.1.suc, Nat.1.suc, matrix_mul[R](Nat.1.suc, Nat.1.suc, Nat.1.suc, a, b), fin_last(Nat.1), fin_zero(Nat.1))
    matrix_entry_mul_two_by_two[R](a, b, fin_zero(Nat.1), fin_zero(Nat.1))
    matrix_entry[R](Nat.1.suc, Nat.1.suc, matrix_mul[R](Nat.1.suc, Nat.1.suc, Nat.1.suc, a, b), fin_zero(Nat.1), fin_zero(Nat.1)) =
        matrix_entry[R](Nat.1.suc, Nat.1.suc, a, fin_zero(Nat.1), fin_zero(Nat.1)) *
            matrix_entry[R](Nat.1.suc, Nat.1.suc, b, fin_zero(Nat.1), fin_zero(Nat.1)) +
        matrix_entry[R](Nat.1.suc, Nat.1.suc, a, fin_zero(Nat.1), fin_last(Nat.1)) *
            matrix_entry[R](Nat.1.suc, Nat.1.suc, b, fin_last(Nat.1), fin_zero(Nat.1))
    matrix_entry_mul_two_by_two[R](a, b, fin_last(Nat.1), fin_last(Nat.1))
    matrix_entry[R](Nat.1.suc, Nat.1.suc, matrix_mul[R](Nat.1.suc, Nat.1.suc, Nat.1.suc, a, b), fin_last(Nat.1), fin_last(Nat.1)) =
        matrix_entry[R](Nat.1.suc, Nat.1.suc, a, fin_last(Nat.1), fin_zero(Nat.1)) *
            matrix_entry[R](Nat.1.suc, Nat.1.suc, b, fin_zero(Nat.1), fin_last(Nat.1)) +
        matrix_entry[R](Nat.1.suc, Nat.1.suc, a, fin_last(Nat.1), fin_last(Nat.1)) *
            matrix_entry[R](Nat.1.suc, Nat.1.suc, b, fin_last(Nat.1), fin_last(Nat.1))
    matrix_entry_mul_two_by_two[R](a, b, fin_zero(Nat.1), fin_last(Nat.1))
    matrix_entry[R](Nat.1.suc, Nat.1.suc, matrix_mul[R](Nat.1.suc, Nat.1.suc, Nat.1.suc, a, b), fin_zero(Nat.1), fin_last(Nat.1)) =
        matrix_entry[R](Nat.1.suc, Nat.1.suc, a, fin_zero(Nat.1), fin_zero(Nat.1)) *
            matrix_entry[R](Nat.1.suc, Nat.1.suc, b, fin_zero(Nat.1), fin_last(Nat.1)) +
        matrix_entry[R](Nat.1.suc, Nat.1.suc, a, fin_zero(Nat.1), fin_last(Nat.1)) *
            matrix_entry[R](Nat.1.suc, Nat.1.suc, b, fin_last(Nat.1), fin_last(Nat.1))
    matrix_entry_mul_two_by_two[R](a, b, fin_last(Nat.1), fin_zero(Nat.1))
    matrix_entry[R](Nat.1.suc, Nat.1.suc, matrix_mul[R](Nat.1.suc, Nat.1.suc, Nat.1.suc, a, b), fin_last(Nat.1), fin_zero(Nat.1)) =
        matrix_entry[R](Nat.1.suc, Nat.1.suc, a, fin_last(Nat.1), fin_zero(Nat.1)) *
            matrix_entry[R](Nat.1.suc, Nat.1.suc, b, fin_zero(Nat.1), fin_zero(Nat.1)) +
        matrix_entry[R](Nat.1.suc, Nat.1.suc, a, fin_last(Nat.1), fin_last(Nat.1)) *
            matrix_entry[R](Nat.1.suc, Nat.1.suc, b, fin_last(Nat.1), fin_zero(Nat.1))
    det2_mul_identity(
        matrix_entry[R](Nat.1.suc, Nat.1.suc, a, fin_zero(Nat.1), fin_zero(Nat.1)),
        matrix_entry[R](Nat.1.suc, Nat.1.suc, a, fin_zero(Nat.1), fin_last(Nat.1)),
        matrix_entry[R](Nat.1.suc, Nat.1.suc, a, fin_last(Nat.1), fin_zero(Nat.1)),
        matrix_entry[R](Nat.1.suc, Nat.1.suc, a, fin_last(Nat.1), fin_last(Nat.1)),
        matrix_entry[R](Nat.1.suc, Nat.1.suc, b, fin_zero(Nat.1), fin_zero(Nat.1)),
        matrix_entry[R](Nat.1.suc, Nat.1.suc, b, fin_zero(Nat.1), fin_last(Nat.1)),
        matrix_entry[R](Nat.1.suc, Nat.1.suc, b, fin_last(Nat.1), fin_zero(Nat.1)),
        matrix_entry[R](Nat.1.suc, Nat.1.suc, b, fin_last(Nat.1), fin_last(Nat.1)))
    matrix_det_suc[R](Nat.1, matrix_mul[R](Nat.1.suc, Nat.1.suc, Nat.1.suc, a, b)) =
        (matrix_entry[R](Nat.1.suc, Nat.1.suc, a, fin_zero(Nat.1), fin_zero(Nat.1)) *
            matrix_entry[R](Nat.1.suc, Nat.1.suc, a, fin_last(Nat.1), fin_last(Nat.1)) -
            matrix_entry[R](Nat.1.suc, Nat.1.suc, a, fin_zero(Nat.1), fin_last(Nat.1)) *
                matrix_entry[R](Nat.1.suc, Nat.1.suc, a, fin_last(Nat.1), fin_zero(Nat.1))) *
        (matrix_entry[R](Nat.1.suc, Nat.1.suc, b, fin_zero(Nat.1), fin_zero(Nat.1)) *
            matrix_entry[R](Nat.1.suc, Nat.1.suc, b, fin_last(Nat.1), fin_last(Nat.1)) -
            matrix_entry[R](Nat.1.suc, Nat.1.suc, b, fin_zero(Nat.1), fin_last(Nat.1)) *
                matrix_entry[R](Nat.1.suc, Nat.1.suc, b, fin_last(Nat.1), fin_zero(Nat.1)))
    matrix_det_suc_two_by_two[R](a)
    matrix_det_suc_two_by_two[R](b)
    matrix_det_suc[R](Nat.1, a) * matrix_det_suc[R](Nat.1, b) =
        (matrix_entry[R](Nat.1.suc, Nat.1.suc, a, fin_zero(Nat.1), fin_zero(Nat.1)) *
            matrix_entry[R](Nat.1.suc, Nat.1.suc, a, fin_last(Nat.1), fin_last(Nat.1)) -
            matrix_entry[R](Nat.1.suc, Nat.1.suc, a, fin_zero(Nat.1), fin_last(Nat.1)) *
                matrix_entry[R](Nat.1.suc, Nat.1.suc, a, fin_last(Nat.1), fin_zero(Nat.1))) *
        (matrix_entry[R](Nat.1.suc, Nat.1.suc, b, fin_zero(Nat.1), fin_zero(Nat.1)) *
            matrix_entry[R](Nat.1.suc, Nat.1.suc, b, fin_last(Nat.1), fin_last(Nat.1)) -
            matrix_entry[R](Nat.1.suc, Nat.1.suc, b, fin_zero(Nat.1), fin_last(Nat.1)) *
                matrix_entry[R](Nat.1.suc, Nat.1.suc, b, fin_last(Nat.1), fin_zero(Nat.1)))
    matrix_det_suc[R](Nat.1, matrix_mul[R](Nat.1.suc, Nat.1.suc, Nat.1.suc, a, b)) =
        matrix_det_suc[R](Nat.1, a) * matrix_det_suc[R](Nat.1, b)
}

/// A determinant-list whose first row is zero vanishes.
theorem determinant_list_zero_row_head[R: Ring, I](entry: (I, I) -> R, row: I, tail: List[I], cols: List[I]) {
    (forall(col: I) { entry(row, col) = R.0 }) implies
        determinant_list(entry, List.cons(row, tail), cols) = R.0
} by {
    if forall(col: I) { entry(row, col) = R.0 } {
        determinant_list_cons_rows(entry, row, tail, cols)
        determinant_list(entry, List.cons(row, tail), cols) = sum[R](map[I, R](cols, function(col: I) {
            entry(row, col) * alternating_sign[R](list_index_or_zero(cols, col)) *
                determinant_list(entry, tail, cols.remove_one(col))
        }))
        forall(col: I) {
            entry(row, col) = R.0
            entry(row, col) * alternating_sign[R](list_index_or_zero(cols, col)) *
                determinant_list(entry, tail, cols.remove_one(col)) = R.0
        }
        sum_map_zero_ring[I, R](cols, function(col: I) {
            entry(row, col) * alternating_sign[R](list_index_or_zero(cols, col)) *
                determinant_list(entry, tail, cols.remove_one(col))
        })
        sum[R](map[I, R](cols, function(col: I) {
            entry(row, col) * alternating_sign[R](list_index_or_zero(cols, col)) *
                determinant_list(entry, tail, cols.remove_one(col))
        })) = R.0
        determinant_list(entry, List.cons(row, tail), cols) = R.0
    }
}

/// A determinant-list containing a zero row vanishes.
theorem determinant_list_zero_row_anywhere[R: Ring, I](entry: (I, I) -> R, rows: List[I], cols: List[I], row: I) {
    rows.contains(row) and (forall(col: I) { entry(row, col) = R.0 }) implies
        determinant_list(entry, rows, cols) = R.0
} by {
    define p(l: List[I]) -> Bool {
        forall(other: List[I]) {
            l.contains(row) and (forall(col: I) { entry(row, col) = R.0 }) implies
                determinant_list(entry, l, other) = R.0
        }
    }
    forall(other: List[I]) {
        if List.nil[I].contains(row) and (forall(col: I) { entry(row, col) = R.0 }) {
            List.nil[I].contains(row) = false
            false
        }
        p(List.nil[I])
    }
    forall(head: I, tail: List[I]) {
        if p(tail) {
            forall(other2: List[I]) {
                if List.cons(head, tail).contains(row) and (forall(col: I) { entry(row, col) = R.0 }) {
                    if head = row {
                        determinant_list_zero_row_head(entry, row, tail, other2)
                        determinant_list(entry, List.cons(head, tail), other2) = R.0
                    } else {
                        if tail.contains(row) and (forall(col: I) { entry(row, col) = R.0 }) {
                            determinant_list(entry, tail, other2) = R.0
                            determinant_list_cons_rows(entry, head, tail, other2)
                            determinant_list(entry, List.cons(head, tail), other2) = sum[R](map[I, R](other2, function(col: I) {
                                entry(head, col) * alternating_sign[R](list_index_or_zero(other2, col)) *
                                    determinant_list(entry, tail, other2.remove_one(col))
                            }))
                            forall(col2: I) {
                                if tail.contains(row) and (forall(col3: I) { entry(row, col3) = R.0 }) {
                                    determinant_list(entry, tail, other2.remove_one(col2)) = R.0
                                    entry(head, col2) * alternating_sign[R](list_index_or_zero(other2, col2)) *
                                        determinant_list(entry, tail, other2.remove_one(col2)) = R.0
                                }
                                entry(head, col2) * alternating_sign[R](list_index_or_zero(other2, col2)) *
                                    determinant_list(entry, tail, other2.remove_one(col2)) = R.0
                            }
                            sum_map_zero_ring[I, R](other2, function(col2: I) {
                                entry(head, col2) * alternating_sign[R](list_index_or_zero(other2, col2)) *
                                    determinant_list(entry, tail, other2.remove_one(col2))
                            })
                            sum[R](map[I, R](other2, function(col2: I) {
                                entry(head, col2) * alternating_sign[R](list_index_or_zero(other2, col2)) *
                                    determinant_list(entry, tail, other2.remove_one(col2))
                            })) = R.0
                            determinant_list(entry, List.cons(head, tail), other2) = R.0
                        }
                        determinant_list(entry, List.cons(head, tail), other2) = R.0
                    }
                }
            }
            p(List.cons(head, tail))
        }
    }
    p(List.nil[I]) and forall(head: I, tail: List[I]) { p(tail) implies p(List.cons(head, tail)) }
    List.induction(p)
    forall(l: List[I]) { p(l) }
    p(rows)
    if rows.contains(row) and (forall(col: I) { entry(row, col) = R.0 }) {
        determinant_list(entry, rows, cols) = R.0
    }
}

/// The canonical determinant of a matrix with a zero row is zero.
theorem matrix_det_suc_zero_row[R: Ring](n: Nat, a: Matrix[R, n.suc, n.suc], row: Fin[n.suc]) {
    (forall(col: Fin[n.suc]) { matrix_entry[R](n.suc, n.suc, a, row, col) = R.0 }) implies
        matrix_det_suc[R](n, a) = R.0
} by {
    if forall(col: Fin[n.suc]) { matrix_entry[R](n.suc, n.suc, a, row, col) = R.0 } {
        matrix_det_suc_unfold[R](n, a)
        matrix_det_suc[R](n, a) = matrix_det_list[R](n.suc, a, fin_enum_suc(n), fin_enum_suc(n))
        forall(col: Fin[n.suc]) {
            matrix_entry[R](n.suc, n.suc, a, row, col) = R.0
            square_matrix_entry_function[R](n.suc, a, row, col) =
                matrix_entry[R](n.suc, n.suc, a, row, col)
            square_matrix_entry_function[R](n.suc, a, row, col) = R.0
        }
        matrix_det_suc_enum_contains(n, row)
        fin_enum_suc(n).contains(row)
        determinant_list_zero_row_anywhere[R, Fin[n.suc]](square_matrix_entry_function[R](n.suc, a),
            fin_enum_suc(n), fin_enum_suc(n), row)
        determinant_list[R, Fin[n.suc]](square_matrix_entry_function[R](n.suc, a),
            fin_enum_suc(n), fin_enum_suc(n)) = R.0
        matrix_det_list[R](n.suc, a, fin_enum_suc(n), fin_enum_suc(n)) = R.0
        matrix_det_suc[R](n, a) = R.0
    }
}

/// The canonical determinant of the `2 x 2` identity matrix is one.
theorem matrix_det_suc_two_by_two_identity[R: Ring] {
    matrix_det_suc[R](Nat.1, matrix_one[R](Nat.1.suc)) = R.1
} by {
    matrix_det_suc_two_by_two[R](matrix_one[R](Nat.1.suc))
    matrix_det_suc[R](Nat.1, matrix_one[R](Nat.1.suc)) =
        matrix_entry[R](Nat.1.suc, Nat.1.suc, matrix_one[R](Nat.1.suc), fin_zero(Nat.1), fin_zero(Nat.1)) *
            matrix_entry[R](Nat.1.suc, Nat.1.suc, matrix_one[R](Nat.1.suc), fin_last(Nat.1), fin_last(Nat.1)) -
        matrix_entry[R](Nat.1.suc, Nat.1.suc, matrix_one[R](Nat.1.suc), fin_zero(Nat.1), fin_last(Nat.1)) *
            matrix_entry[R](Nat.1.suc, Nat.1.suc, matrix_one[R](Nat.1.suc), fin_last(Nat.1), fin_zero(Nat.1))
    matrix_entry_one_diag[R](Nat.1.suc, fin_zero(Nat.1))
    matrix_entry[R](Nat.1.suc, Nat.1.suc, matrix_one[R](Nat.1.suc), fin_zero(Nat.1), fin_zero(Nat.1)) = R.1
    matrix_entry_one_diag[R](Nat.1.suc, fin_last(Nat.1))
    matrix_entry[R](Nat.1.suc, Nat.1.suc, matrix_one[R](Nat.1.suc), fin_last(Nat.1), fin_last(Nat.1)) = R.1
    fin_zero_value(Nat.1)
    fin_zero(Nat.1).value = Nat.0
    fin_last_value(Nat.1)
    fin_last(Nat.1).value = Nat.1
    Nat.0 != Nat.1
    if fin_zero(Nat.1) = fin_last(Nat.1) {
        fin_zero(Nat.1).value = fin_last(Nat.1).value
        Nat.0 = Nat.1
        false
    }
    not fin_zero(Nat.1) = fin_last(Nat.1)
    fin_zero(Nat.1) != fin_last(Nat.1)
    matrix_entry_one_off_diag[R](Nat.1.suc, fin_zero(Nat.1), fin_last(Nat.1))
    matrix_entry[R](Nat.1.suc, Nat.1.suc, matrix_one[R](Nat.1.suc), fin_zero(Nat.1), fin_last(Nat.1)) = R.0
    matrix_entry_one_off_diag[R](Nat.1.suc, fin_last(Nat.1), fin_zero(Nat.1))
    matrix_entry[R](Nat.1.suc, Nat.1.suc, matrix_one[R](Nat.1.suc), fin_last(Nat.1), fin_zero(Nat.1)) = R.0
    matrix_det_suc[R](Nat.1, matrix_one[R](Nat.1.suc)) = R.1 * R.1 - R.0 * R.0
    R.1 * R.1 = R.1
    R.0 * R.0 = R.0
    R.1 - R.0 = R.1
    R.1 * R.1 - R.0 * R.0 = R.1 - R.0
    R.1 * R.1 - R.0 * R.0 = R.1
    matrix_det_suc[R](Nat.1, matrix_one[R](Nat.1.suc)) = R.1
}


/// Subtracting zero changes nothing.
theorem sub_zero_right[R: Ring](x: R) {
    x - R.0 = x
} by {
}

theorem matrix_det_suc_two_by_two_transpose[R: CommRing](a: Matrix[R, Nat.1.suc, Nat.1.suc]) {
    matrix_det_suc[R](Nat.1, matrix_transpose[R](Nat.1.suc, Nat.1.suc, a)) = matrix_det_suc[R](Nat.1, a)
} by {
    matrix_det_suc_two_by_two[R](matrix_transpose[R](Nat.1.suc, Nat.1.suc, a))
    matrix_det_suc[R](Nat.1, matrix_transpose[R](Nat.1.suc, Nat.1.suc, a)) =
        matrix_entry[R](Nat.1.suc, Nat.1.suc, matrix_transpose[R](Nat.1.suc, Nat.1.suc, a), fin_zero(Nat.1), fin_zero(Nat.1)) *
            matrix_entry[R](Nat.1.suc, Nat.1.suc, matrix_transpose[R](Nat.1.suc, Nat.1.suc, a), fin_last(Nat.1), fin_last(Nat.1)) -
        matrix_entry[R](Nat.1.suc, Nat.1.suc, matrix_transpose[R](Nat.1.suc, Nat.1.suc, a), fin_zero(Nat.1), fin_last(Nat.1)) *
            matrix_entry[R](Nat.1.suc, Nat.1.suc, matrix_transpose[R](Nat.1.suc, Nat.1.suc, a), fin_last(Nat.1), fin_zero(Nat.1))
    matrix_det_suc_two_by_two[R](a)
    matrix_det_suc[R](Nat.1, a) =
        matrix_entry[R](Nat.1.suc, Nat.1.suc, a, fin_zero(Nat.1), fin_zero(Nat.1)) *
            matrix_entry[R](Nat.1.suc, Nat.1.suc, a, fin_last(Nat.1), fin_last(Nat.1)) -
        matrix_entry[R](Nat.1.suc, Nat.1.suc, a, fin_zero(Nat.1), fin_last(Nat.1)) *
            matrix_entry[R](Nat.1.suc, Nat.1.suc, a, fin_last(Nat.1), fin_zero(Nat.1))
    matrix_entry_transpose[R](Nat.1.suc, Nat.1.suc, a, fin_zero(Nat.1), fin_zero(Nat.1))
    matrix_entry[R](Nat.1.suc, Nat.1.suc, matrix_transpose[R](Nat.1.suc, Nat.1.suc, a), fin_zero(Nat.1), fin_zero(Nat.1)) =
        matrix_entry[R](Nat.1.suc, Nat.1.suc, a, fin_zero(Nat.1), fin_zero(Nat.1))
    matrix_entry_transpose[R](Nat.1.suc, Nat.1.suc, a, fin_last(Nat.1), fin_last(Nat.1))
    matrix_entry[R](Nat.1.suc, Nat.1.suc, matrix_transpose[R](Nat.1.suc, Nat.1.suc, a), fin_last(Nat.1), fin_last(Nat.1)) =
        matrix_entry[R](Nat.1.suc, Nat.1.suc, a, fin_last(Nat.1), fin_last(Nat.1))
    matrix_entry_transpose[R](Nat.1.suc, Nat.1.suc, a, fin_zero(Nat.1), fin_last(Nat.1))
    matrix_entry[R](Nat.1.suc, Nat.1.suc, matrix_transpose[R](Nat.1.suc, Nat.1.suc, a), fin_last(Nat.1), fin_zero(Nat.1)) =
        matrix_entry[R](Nat.1.suc, Nat.1.suc, a, fin_zero(Nat.1), fin_last(Nat.1))
    matrix_entry_transpose[R](Nat.1.suc, Nat.1.suc, a, fin_last(Nat.1), fin_zero(Nat.1))
    matrix_entry[R](Nat.1.suc, Nat.1.suc, matrix_transpose[R](Nat.1.suc, Nat.1.suc, a), fin_zero(Nat.1), fin_last(Nat.1)) =
        matrix_entry[R](Nat.1.suc, Nat.1.suc, a, fin_last(Nat.1), fin_zero(Nat.1))
    matrix_entry[R](Nat.1.suc, Nat.1.suc, a, fin_last(Nat.1), fin_zero(Nat.1)) *
        matrix_entry[R](Nat.1.suc, Nat.1.suc, a, fin_zero(Nat.1), fin_last(Nat.1)) =
        matrix_entry[R](Nat.1.suc, Nat.1.suc, a, fin_zero(Nat.1), fin_last(Nat.1)) *
            matrix_entry[R](Nat.1.suc, Nat.1.suc, a, fin_last(Nat.1), fin_zero(Nat.1))
    matrix_entry[R](Nat.1.suc, Nat.1.suc, matrix_transpose[R](Nat.1.suc, Nat.1.suc, a), fin_zero(Nat.1), fin_last(Nat.1)) *
        matrix_entry[R](Nat.1.suc, Nat.1.suc, matrix_transpose[R](Nat.1.suc, Nat.1.suc, a), fin_last(Nat.1), fin_zero(Nat.1)) =
        matrix_entry[R](Nat.1.suc, Nat.1.suc, a, fin_zero(Nat.1), fin_last(Nat.1)) *
            matrix_entry[R](Nat.1.suc, Nat.1.suc, a, fin_last(Nat.1), fin_zero(Nat.1))
    matrix_det_suc[R](Nat.1, matrix_transpose[R](Nat.1.suc, Nat.1.suc, a)) =
        matrix_entry[R](Nat.1.suc, Nat.1.suc, a, fin_zero(Nat.1), fin_zero(Nat.1)) *
            matrix_entry[R](Nat.1.suc, Nat.1.suc, a, fin_last(Nat.1), fin_last(Nat.1)) -
        matrix_entry[R](Nat.1.suc, Nat.1.suc, a, fin_zero(Nat.1), fin_last(Nat.1)) *
            matrix_entry[R](Nat.1.suc, Nat.1.suc, a, fin_last(Nat.1), fin_zero(Nat.1))
    matrix_det_suc[R](Nat.1, matrix_transpose[R](Nat.1.suc, Nat.1.suc, a)) = matrix_det_suc[R](Nat.1, a)
}

theorem matrix_det_suc_two_by_two_swap_rows[R: CommRing](a: Matrix[R, Nat.1.suc, Nat.1.suc]) {
    matrix_det_suc[R](Nat.1, matrix_swap_rows[R](Nat.1.suc, Nat.1.suc, a, fin_zero(Nat.1), fin_last(Nat.1))) =
        -matrix_det_suc[R](Nat.1, a)
} by {
    matrix_det_suc_two_by_two[R](matrix_swap_rows[R](Nat.1.suc, Nat.1.suc, a, fin_zero(Nat.1), fin_last(Nat.1)))
    matrix_det_suc[R](Nat.1, matrix_swap_rows[R](Nat.1.suc, Nat.1.suc, a, fin_zero(Nat.1), fin_last(Nat.1))) =
        matrix_entry[R](Nat.1.suc, Nat.1.suc, matrix_swap_rows[R](Nat.1.suc, Nat.1.suc, a, fin_zero(Nat.1), fin_last(Nat.1)), fin_zero(Nat.1), fin_zero(Nat.1)) *
            matrix_entry[R](Nat.1.suc, Nat.1.suc, matrix_swap_rows[R](Nat.1.suc, Nat.1.suc, a, fin_zero(Nat.1), fin_last(Nat.1)), fin_last(Nat.1), fin_last(Nat.1)) -
        matrix_entry[R](Nat.1.suc, Nat.1.suc, matrix_swap_rows[R](Nat.1.suc, Nat.1.suc, a, fin_zero(Nat.1), fin_last(Nat.1)), fin_zero(Nat.1), fin_last(Nat.1)) *
            matrix_entry[R](Nat.1.suc, Nat.1.suc, matrix_swap_rows[R](Nat.1.suc, Nat.1.suc, a, fin_zero(Nat.1), fin_last(Nat.1)), fin_last(Nat.1), fin_zero(Nat.1))
    matrix_entry_swap_rows[R](Nat.1.suc, Nat.1.suc, a, fin_zero(Nat.1), fin_last(Nat.1), fin_zero(Nat.1), fin_zero(Nat.1))
    matrix_entry[R](Nat.1.suc, Nat.1.suc, matrix_swap_rows[R](Nat.1.suc, Nat.1.suc, a, fin_zero(Nat.1), fin_last(Nat.1)), fin_zero(Nat.1), fin_zero(Nat.1)) =
        matrix_swap_rows_entry[R](Nat.1.suc, Nat.1.suc, a, fin_zero(Nat.1), fin_last(Nat.1), fin_zero(Nat.1), fin_zero(Nat.1))
    matrix_swap_rows_entry[R](Nat.1.suc, Nat.1.suc, a, fin_zero(Nat.1), fin_last(Nat.1), fin_zero(Nat.1), fin_zero(Nat.1)) =
        matrix_entry[R](Nat.1.suc, Nat.1.suc, a, fin_last(Nat.1), fin_zero(Nat.1))
    matrix_entry[R](Nat.1.suc, Nat.1.suc, matrix_swap_rows[R](Nat.1.suc, Nat.1.suc, a, fin_zero(Nat.1), fin_last(Nat.1)), fin_zero(Nat.1), fin_zero(Nat.1)) =
        matrix_entry[R](Nat.1.suc, Nat.1.suc, a, fin_last(Nat.1), fin_zero(Nat.1))
    matrix_entry_swap_rows[R](Nat.1.suc, Nat.1.suc, a, fin_zero(Nat.1), fin_last(Nat.1), fin_last(Nat.1), fin_last(Nat.1))
    matrix_entry[R](Nat.1.suc, Nat.1.suc, matrix_swap_rows[R](Nat.1.suc, Nat.1.suc, a, fin_zero(Nat.1), fin_last(Nat.1)), fin_last(Nat.1), fin_last(Nat.1)) =
        matrix_swap_rows_entry[R](Nat.1.suc, Nat.1.suc, a, fin_zero(Nat.1), fin_last(Nat.1), fin_last(Nat.1), fin_last(Nat.1))
    matrix_swap_rows_entry[R](Nat.1.suc, Nat.1.suc, a, fin_zero(Nat.1), fin_last(Nat.1), fin_last(Nat.1), fin_last(Nat.1)) =
        matrix_entry[R](Nat.1.suc, Nat.1.suc, a, fin_zero(Nat.1), fin_last(Nat.1))
    matrix_entry[R](Nat.1.suc, Nat.1.suc, matrix_swap_rows[R](Nat.1.suc, Nat.1.suc, a, fin_zero(Nat.1), fin_last(Nat.1)), fin_last(Nat.1), fin_last(Nat.1)) =
        matrix_entry[R](Nat.1.suc, Nat.1.suc, a, fin_zero(Nat.1), fin_last(Nat.1))
    matrix_entry_swap_rows[R](Nat.1.suc, Nat.1.suc, a, fin_zero(Nat.1), fin_last(Nat.1), fin_zero(Nat.1), fin_last(Nat.1))
    matrix_entry[R](Nat.1.suc, Nat.1.suc, matrix_swap_rows[R](Nat.1.suc, Nat.1.suc, a, fin_zero(Nat.1), fin_last(Nat.1)), fin_zero(Nat.1), fin_last(Nat.1)) =
        matrix_swap_rows_entry[R](Nat.1.suc, Nat.1.suc, a, fin_zero(Nat.1), fin_last(Nat.1), fin_zero(Nat.1), fin_last(Nat.1))
    matrix_swap_rows_entry[R](Nat.1.suc, Nat.1.suc, a, fin_zero(Nat.1), fin_last(Nat.1), fin_zero(Nat.1), fin_last(Nat.1)) =
        matrix_entry[R](Nat.1.suc, Nat.1.suc, a, fin_last(Nat.1), fin_last(Nat.1))
    matrix_entry[R](Nat.1.suc, Nat.1.suc, matrix_swap_rows[R](Nat.1.suc, Nat.1.suc, a, fin_zero(Nat.1), fin_last(Nat.1)), fin_zero(Nat.1), fin_last(Nat.1)) =
        matrix_entry[R](Nat.1.suc, Nat.1.suc, a, fin_last(Nat.1), fin_last(Nat.1))
    matrix_entry_swap_rows[R](Nat.1.suc, Nat.1.suc, a, fin_zero(Nat.1), fin_last(Nat.1), fin_last(Nat.1), fin_zero(Nat.1))
    matrix_entry[R](Nat.1.suc, Nat.1.suc, matrix_swap_rows[R](Nat.1.suc, Nat.1.suc, a, fin_zero(Nat.1), fin_last(Nat.1)), fin_last(Nat.1), fin_zero(Nat.1)) =
        matrix_swap_rows_entry[R](Nat.1.suc, Nat.1.suc, a, fin_zero(Nat.1), fin_last(Nat.1), fin_last(Nat.1), fin_zero(Nat.1))
    matrix_swap_rows_entry[R](Nat.1.suc, Nat.1.suc, a, fin_zero(Nat.1), fin_last(Nat.1), fin_last(Nat.1), fin_zero(Nat.1)) =
        matrix_entry[R](Nat.1.suc, Nat.1.suc, a, fin_zero(Nat.1), fin_zero(Nat.1))
    matrix_entry[R](Nat.1.suc, Nat.1.suc, matrix_swap_rows[R](Nat.1.suc, Nat.1.suc, a, fin_zero(Nat.1), fin_last(Nat.1)), fin_last(Nat.1), fin_zero(Nat.1)) =
        matrix_entry[R](Nat.1.suc, Nat.1.suc, a, fin_zero(Nat.1), fin_zero(Nat.1))
    matrix_det_suc_two_by_two[R](a)
    matrix_det_suc[R](Nat.1, a) =
        matrix_entry[R](Nat.1.suc, Nat.1.suc, a, fin_zero(Nat.1), fin_zero(Nat.1)) *
            matrix_entry[R](Nat.1.suc, Nat.1.suc, a, fin_last(Nat.1), fin_last(Nat.1)) -
        matrix_entry[R](Nat.1.suc, Nat.1.suc, a, fin_zero(Nat.1), fin_last(Nat.1)) *
            matrix_entry[R](Nat.1.suc, Nat.1.suc, a, fin_last(Nat.1), fin_zero(Nat.1))
    matrix_entry[R](Nat.1.suc, Nat.1.suc, a, fin_last(Nat.1), fin_zero(Nat.1)) *
        matrix_entry[R](Nat.1.suc, Nat.1.suc, a, fin_zero(Nat.1), fin_last(Nat.1)) -
        matrix_entry[R](Nat.1.suc, Nat.1.suc, a, fin_last(Nat.1), fin_last(Nat.1)) *
            matrix_entry[R](Nat.1.suc, Nat.1.suc, a, fin_zero(Nat.1), fin_zero(Nat.1)) =
        -(matrix_entry[R](Nat.1.suc, Nat.1.suc, a, fin_zero(Nat.1), fin_zero(Nat.1)) *
            matrix_entry[R](Nat.1.suc, Nat.1.suc, a, fin_last(Nat.1), fin_last(Nat.1)) -
            matrix_entry[R](Nat.1.suc, Nat.1.suc, a, fin_zero(Nat.1), fin_last(Nat.1)) *
                matrix_entry[R](Nat.1.suc, Nat.1.suc, a, fin_last(Nat.1), fin_zero(Nat.1)))
    matrix_det_suc[R](Nat.1, matrix_swap_rows[R](Nat.1.suc, Nat.1.suc, a, fin_zero(Nat.1), fin_last(Nat.1))) =
        -(matrix_entry[R](Nat.1.suc, Nat.1.suc, a, fin_zero(Nat.1), fin_zero(Nat.1)) *
            matrix_entry[R](Nat.1.suc, Nat.1.suc, a, fin_last(Nat.1), fin_last(Nat.1)) -
            matrix_entry[R](Nat.1.suc, Nat.1.suc, a, fin_zero(Nat.1), fin_last(Nat.1)) *
                matrix_entry[R](Nat.1.suc, Nat.1.suc, a, fin_last(Nat.1), fin_zero(Nat.1)))
    matrix_det_suc[R](Nat.1, matrix_swap_rows[R](Nat.1.suc, Nat.1.suc, a, fin_zero(Nat.1), fin_last(Nat.1))) =
        -matrix_det_suc[R](Nat.1, a)
}

theorem matrix_det_suc_two_by_two_equal_rows[R: CommRing](a: Matrix[R, Nat.1.suc, Nat.1.suc]) {
    (forall(col: Fin[Nat.1.suc]) {
        matrix_entry[R](Nat.1.suc, Nat.1.suc, a, fin_zero(Nat.1), col) =
            matrix_entry[R](Nat.1.suc, Nat.1.suc, a, fin_last(Nat.1), col)
    }) implies matrix_det_suc[R](Nat.1, a) = R.0
} by {
    if forall(col: Fin[Nat.1.suc]) {
        matrix_entry[R](Nat.1.suc, Nat.1.suc, a, fin_zero(Nat.1), col) =
            matrix_entry[R](Nat.1.suc, Nat.1.suc, a, fin_last(Nat.1), col)
    } {
        matrix_entry[R](Nat.1.suc, Nat.1.suc, a, fin_zero(Nat.1), fin_zero(Nat.1)) =
            matrix_entry[R](Nat.1.suc, Nat.1.suc, a, fin_last(Nat.1), fin_zero(Nat.1))
        matrix_entry[R](Nat.1.suc, Nat.1.suc, a, fin_zero(Nat.1), fin_last(Nat.1)) =
            matrix_entry[R](Nat.1.suc, Nat.1.suc, a, fin_last(Nat.1), fin_last(Nat.1))
        matrix_det_suc_two_by_two[R](a)
        matrix_det_suc[R](Nat.1, a) =
            matrix_entry[R](Nat.1.suc, Nat.1.suc, a, fin_zero(Nat.1), fin_zero(Nat.1)) *
                matrix_entry[R](Nat.1.suc, Nat.1.suc, a, fin_last(Nat.1), fin_last(Nat.1)) -
            matrix_entry[R](Nat.1.suc, Nat.1.suc, a, fin_zero(Nat.1), fin_last(Nat.1)) *
                matrix_entry[R](Nat.1.suc, Nat.1.suc, a, fin_last(Nat.1), fin_zero(Nat.1))
        matrix_entry[R](Nat.1.suc, Nat.1.suc, a, fin_zero(Nat.1), fin_zero(Nat.1)) *
            matrix_entry[R](Nat.1.suc, Nat.1.suc, a, fin_last(Nat.1), fin_last(Nat.1)) =
            matrix_entry[R](Nat.1.suc, Nat.1.suc, a, fin_zero(Nat.1), fin_zero(Nat.1)) *
                matrix_entry[R](Nat.1.suc, Nat.1.suc, a, fin_zero(Nat.1), fin_last(Nat.1))
        matrix_entry[R](Nat.1.suc, Nat.1.suc, a, fin_zero(Nat.1), fin_last(Nat.1)) *
            matrix_entry[R](Nat.1.suc, Nat.1.suc, a, fin_last(Nat.1), fin_zero(Nat.1)) =
            matrix_entry[R](Nat.1.suc, Nat.1.suc, a, fin_zero(Nat.1), fin_last(Nat.1)) *
                matrix_entry[R](Nat.1.suc, Nat.1.suc, a, fin_zero(Nat.1), fin_zero(Nat.1))
        matrix_entry[R](Nat.1.suc, Nat.1.suc, a, fin_zero(Nat.1), fin_zero(Nat.1)) *
            matrix_entry[R](Nat.1.suc, Nat.1.suc, a, fin_zero(Nat.1), fin_last(Nat.1)) =
            matrix_entry[R](Nat.1.suc, Nat.1.suc, a, fin_zero(Nat.1), fin_last(Nat.1)) *
                matrix_entry[R](Nat.1.suc, Nat.1.suc, a, fin_zero(Nat.1), fin_zero(Nat.1))
        matrix_entry[R](Nat.1.suc, Nat.1.suc, a, fin_zero(Nat.1), fin_zero(Nat.1)) *
            matrix_entry[R](Nat.1.suc, Nat.1.suc, a, fin_last(Nat.1), fin_last(Nat.1)) =
            matrix_entry[R](Nat.1.suc, Nat.1.suc, a, fin_zero(Nat.1), fin_last(Nat.1)) *
                matrix_entry[R](Nat.1.suc, Nat.1.suc, a, fin_last(Nat.1), fin_zero(Nat.1))
        matrix_entry[R](Nat.1.suc, Nat.1.suc, a, fin_zero(Nat.1), fin_zero(Nat.1)) *
            matrix_entry[R](Nat.1.suc, Nat.1.suc, a, fin_last(Nat.1), fin_last(Nat.1)) -
            matrix_entry[R](Nat.1.suc, Nat.1.suc, a, fin_zero(Nat.1), fin_last(Nat.1)) *
                matrix_entry[R](Nat.1.suc, Nat.1.suc, a, fin_last(Nat.1), fin_zero(Nat.1)) =
            matrix_entry[R](Nat.1.suc, Nat.1.suc, a, fin_zero(Nat.1), fin_zero(Nat.1)) *
                matrix_entry[R](Nat.1.suc, Nat.1.suc, a, fin_last(Nat.1), fin_last(Nat.1)) -
                matrix_entry[R](Nat.1.suc, Nat.1.suc, a, fin_zero(Nat.1), fin_zero(Nat.1)) *
                    matrix_entry[R](Nat.1.suc, Nat.1.suc, a, fin_last(Nat.1), fin_last(Nat.1))
        matrix_entry[R](Nat.1.suc, Nat.1.suc, a, fin_zero(Nat.1), fin_zero(Nat.1)) *
            matrix_entry[R](Nat.1.suc, Nat.1.suc, a, fin_last(Nat.1), fin_last(Nat.1)) -
            matrix_entry[R](Nat.1.suc, Nat.1.suc, a, fin_zero(Nat.1), fin_zero(Nat.1)) *
                matrix_entry[R](Nat.1.suc, Nat.1.suc, a, fin_last(Nat.1), fin_last(Nat.1)) = R.0
        matrix_entry[R](Nat.1.suc, Nat.1.suc, a, fin_zero(Nat.1), fin_zero(Nat.1)) *
            matrix_entry[R](Nat.1.suc, Nat.1.suc, a, fin_last(Nat.1), fin_last(Nat.1)) -
            matrix_entry[R](Nat.1.suc, Nat.1.suc, a, fin_zero(Nat.1), fin_last(Nat.1)) *
                matrix_entry[R](Nat.1.suc, Nat.1.suc, a, fin_last(Nat.1), fin_zero(Nat.1)) = R.0
        matrix_det_suc[R](Nat.1, a) = R.0
    }
}

theorem matrix_det_suc_two_by_two_upper_triangular[R: Ring](a: Matrix[R, Nat.1.suc, Nat.1.suc]) {
    matrix_entry[R](Nat.1.suc, Nat.1.suc, a, fin_last(Nat.1), fin_zero(Nat.1)) = R.0 implies
        matrix_det_suc[R](Nat.1, a) =
            matrix_entry[R](Nat.1.suc, Nat.1.suc, a, fin_zero(Nat.1), fin_zero(Nat.1)) *
                matrix_entry[R](Nat.1.suc, Nat.1.suc, a, fin_last(Nat.1), fin_last(Nat.1))
} by {
    if matrix_entry[R](Nat.1.suc, Nat.1.suc, a, fin_last(Nat.1), fin_zero(Nat.1)) = R.0 {
        matrix_det_suc_two_by_two[R](a)
        matrix_det_suc[R](Nat.1, a) =
            matrix_entry[R](Nat.1.suc, Nat.1.suc, a, fin_zero(Nat.1), fin_zero(Nat.1)) *
                matrix_entry[R](Nat.1.suc, Nat.1.suc, a, fin_last(Nat.1), fin_last(Nat.1)) -
            matrix_entry[R](Nat.1.suc, Nat.1.suc, a, fin_zero(Nat.1), fin_last(Nat.1)) *
                matrix_entry[R](Nat.1.suc, Nat.1.suc, a, fin_last(Nat.1), fin_zero(Nat.1))
        matrix_entry[R](Nat.1.suc, Nat.1.suc, a, fin_zero(Nat.1), fin_last(Nat.1)) *
            matrix_entry[R](Nat.1.suc, Nat.1.suc, a, fin_last(Nat.1), fin_zero(Nat.1)) =
            matrix_entry[R](Nat.1.suc, Nat.1.suc, a, fin_zero(Nat.1), fin_last(Nat.1)) * R.0
        matrix_entry[R](Nat.1.suc, Nat.1.suc, a, fin_zero(Nat.1), fin_last(Nat.1)) * R.0 = R.0
        matrix_entry[R](Nat.1.suc, Nat.1.suc, a, fin_zero(Nat.1), fin_last(Nat.1)) *
            matrix_entry[R](Nat.1.suc, Nat.1.suc, a, fin_last(Nat.1), fin_zero(Nat.1)) = R.0
        matrix_entry[R](Nat.1.suc, Nat.1.suc, a, fin_zero(Nat.1), fin_zero(Nat.1)) *
            matrix_entry[R](Nat.1.suc, Nat.1.suc, a, fin_last(Nat.1), fin_last(Nat.1)) -
            matrix_entry[R](Nat.1.suc, Nat.1.suc, a, fin_zero(Nat.1), fin_last(Nat.1)) *
                matrix_entry[R](Nat.1.suc, Nat.1.suc, a, fin_last(Nat.1), fin_zero(Nat.1)) =
            matrix_entry[R](Nat.1.suc, Nat.1.suc, a, fin_zero(Nat.1), fin_zero(Nat.1)) *
                matrix_entry[R](Nat.1.suc, Nat.1.suc, a, fin_last(Nat.1), fin_last(Nat.1)) - R.0
        sub_zero_right[R](matrix_entry[R](Nat.1.suc, Nat.1.suc, a, fin_zero(Nat.1), fin_zero(Nat.1)) *
            matrix_entry[R](Nat.1.suc, Nat.1.suc, a, fin_last(Nat.1), fin_last(Nat.1)))
        matrix_entry[R](Nat.1.suc, Nat.1.suc, a, fin_zero(Nat.1), fin_zero(Nat.1)) *
            matrix_entry[R](Nat.1.suc, Nat.1.suc, a, fin_last(Nat.1), fin_last(Nat.1)) - R.0 =
            matrix_entry[R](Nat.1.suc, Nat.1.suc, a, fin_zero(Nat.1), fin_zero(Nat.1)) *
                matrix_entry[R](Nat.1.suc, Nat.1.suc, a, fin_last(Nat.1), fin_last(Nat.1))
        matrix_entry[R](Nat.1.suc, Nat.1.suc, a, fin_zero(Nat.1), fin_zero(Nat.1)) *
            matrix_entry[R](Nat.1.suc, Nat.1.suc, a, fin_last(Nat.1), fin_last(Nat.1)) -
            matrix_entry[R](Nat.1.suc, Nat.1.suc, a, fin_zero(Nat.1), fin_last(Nat.1)) *
                matrix_entry[R](Nat.1.suc, Nat.1.suc, a, fin_last(Nat.1), fin_zero(Nat.1)) =
            matrix_entry[R](Nat.1.suc, Nat.1.suc, a, fin_zero(Nat.1), fin_zero(Nat.1)) *
                matrix_entry[R](Nat.1.suc, Nat.1.suc, a, fin_last(Nat.1), fin_last(Nat.1))
        matrix_det_suc[R](Nat.1, a) =
            matrix_entry[R](Nat.1.suc, Nat.1.suc, a, fin_zero(Nat.1), fin_zero(Nat.1)) *
                matrix_entry[R](Nat.1.suc, Nat.1.suc, a, fin_last(Nat.1), fin_last(Nat.1))
    }
}

/// The entry function with row and column roles exchanged.
define flip_entry[R: Ring, I](entry: (I, I) -> R, c: I, r: I) -> R {
    entry(r, c)
}

/// The sign identity used when exchanging row and column roles: moving a
/// successor from one sign argument to the other leaves the product unchanged.
/// Alternating signs commute with each other: both are central units.
theorem alternating_sign_mul_comm[R: Ring](a: Nat, b: Nat) {
    alternating_sign[R](a) * alternating_sign[R](b) = alternating_sign[R](b) * alternating_sign[R](a)
} by {
    define p(k: Nat) -> Bool {
        alternating_sign[R](k) * alternating_sign[R](b) = alternating_sign[R](b) * alternating_sign[R](k)
    }
    alternating_sign_zero[R]
    alternating_sign[R](Nat.0) = R.1
    R.1 * alternating_sign[R](b) = alternating_sign[R](b)
    alternating_sign[R](b) * R.1 = alternating_sign[R](b)
    alternating_sign[R](Nat.0) * alternating_sign[R](b) = alternating_sign[R](b) * alternating_sign[R](Nat.0)
    p(Nat.0)
    forall(k: Nat) {
        if p(k) {
            alternating_sign_suc[R](k)
            alternating_sign[R](k.suc) = -alternating_sign[R](k)
            alternating_sign[R](k.suc) * alternating_sign[R](b) = (-alternating_sign[R](k)) * alternating_sign[R](b)
            mul_neg_left[R](alternating_sign[R](k), alternating_sign[R](b))
            (-alternating_sign[R](k)) * alternating_sign[R](b) = -(alternating_sign[R](k) * alternating_sign[R](b))
            alternating_sign[R](k) * alternating_sign[R](b) = alternating_sign[R](b) * alternating_sign[R](k)
            -(alternating_sign[R](k) * alternating_sign[R](b)) = -(alternating_sign[R](b) * alternating_sign[R](k))
            alternating_sign[R](b) * alternating_sign[R](k.suc) = alternating_sign[R](b) * (-alternating_sign[R](k))
            mul_neg_right[R](alternating_sign[R](b), alternating_sign[R](k))
            alternating_sign[R](b) * (-alternating_sign[R](k)) = -(alternating_sign[R](b) * alternating_sign[R](k))
            alternating_sign[R](k.suc) * alternating_sign[R](b) = alternating_sign[R](b) * alternating_sign[R](k.suc)
            p(k.suc)
        }
    }
    p(Nat.0) and forall(k: Nat) { p(k) implies p(k.suc) }
    Nat.induction(p)
    p(a)
    alternating_sign[R](a) * alternating_sign[R](b) = alternating_sign[R](b) * alternating_sign[R](a)
}

/// The sign identity used when exchanging row and column roles: moving a
/// successor from one sign argument to the other leaves the product unchanged.
theorem alternating_sign_suc_comm[R: Ring](a: Nat, b: Nat) {
    alternating_sign[R](a.suc) * alternating_sign[R](b) =
        alternating_sign[R](b.suc) * alternating_sign[R](a)
} by {
    alternating_sign_suc[R](a)
    alternating_sign_suc[R](b)
    alternating_sign[R](a.suc) = -alternating_sign[R](a)
    alternating_sign[R](b.suc) = -alternating_sign[R](b)
    alternating_sign[R](a.suc) * alternating_sign[R](b) = (-alternating_sign[R](a)) * alternating_sign[R](b)
    mul_neg_left[R](alternating_sign[R](a), alternating_sign[R](b))
    (-alternating_sign[R](a)) * alternating_sign[R](b) = -(alternating_sign[R](a) * alternating_sign[R](b))
    alternating_sign[R](b.suc) * alternating_sign[R](a) = (-alternating_sign[R](b)) * alternating_sign[R](a)
    mul_neg_left[R](alternating_sign[R](b), alternating_sign[R](a))
    (-alternating_sign[R](b)) * alternating_sign[R](a) = -(alternating_sign[R](b) * alternating_sign[R](a))
    alternating_sign_mul_comm[R](a, b)
    alternating_sign[R](a) * alternating_sign[R](b) = alternating_sign[R](b) * alternating_sign[R](a)
    -(alternating_sign[R](a) * alternating_sign[R](b)) = -(alternating_sign[R](b) * alternating_sign[R](a))
    alternating_sign[R](a.suc) * alternating_sign[R](b) =
        alternating_sign[R](b.suc) * alternating_sign[R](a)
}


/// A mapped sum of a pointwise-zero function is zero.
theorem sum_map_zero_fn[T, A: AddCommMonoid](items: List[T], f: T -> A) {
    (forall(item: T) { f(item) = A.0 }) implies sum[A](map[T, A](items, f)) = A.0
} by {
    if forall(item: T) { f(item) = A.0 } {
        define p(xs: List[T]) -> Bool {
            sum[A](map[T, A](xs, f)) = A.0
        }
        map[T, A](List.nil[T], f) = List.nil[A]
        sum[A](List.nil[A]) = A.0
        p(List.nil[T])
        forall(head: T, tail: List[T]) {
            if p(tail) {
                f(head) = A.0
                map[T, A](List.cons(head, tail), f) = List.cons(f(head), map[T, A](tail, f))
                sum[A](map[T, A](tail, f)) = A.0
                sum[A](List.cons(f(head), map[T, A](tail, f))) = f(head) + sum[A](map[T, A](tail, f))
                f(head) + sum[A](map[T, A](tail, f)) = A.0
                sum[A](map[T, A](List.cons(head, tail), f)) = A.0
                p(List.cons(head, tail))
            }
            p(tail) implies p(List.cons(head, tail))
        }
        p(List.nil[T]) and forall(head: T, tail: List[T]) { p(tail) implies p(List.cons(head, tail)) }
        List.induction(p)
        forall(xs: List[T]) { p(xs) }
        p(items)
        sum[A](map[T, A](items, f)) = A.0
    }
}

/// Exchanging the order of two finite list sums.
/// The sum of a pointwise sum of two functions splits.
/// The sum of a pointwise sum of two functions splits.
theorem sum_map_add_fn[T, A: AddCommMonoid](items: List[T], f: T -> A, g: T -> A) {
    sum[A](map[T, A](items, function(y: T) { f(y) + g(y) })) =
        sum[A](map[T, A](items, f)) + sum[A](map[T, A](items, g))
} by {
    map_sum_add[T, A](items, f, g)
    sum[A](map[T, A](items, f)) + sum[A](map[T, A](items, g)) =
        sum[A](map[T, A](items, add_fn(f, g)))
    forall(y: T) {
        add_fn(f, g, y) = f(y) + g(y)
    }
    sum_map_of_pointwise[T, A](items, add_fn(f, g), function(y: T) { f(y) + g(y) })
    sum[A](map[T, A](items, add_fn(f, g))) =
        sum[A](map[T, A](items, function(y: T) { f(y) + g(y) }))
    sum[A](map[T, A](items, function(y: T) { f(y) + g(y) })) =
        sum[A](map[T, A](items, f)) + sum[A](map[T, A](items, g))
}


/// Exchanging the order of two finite list sums.
theorem sum_map_sum_exchange[T, A: AddCommMonoid](xs: List[T], ys: List[T], h: T -> T -> A) {
    sum[A](map[T, A](xs, function(x: T) {
        sum[A](map[T, A](ys, function(y: T) { h(x, y) }))
    })) = sum[A](map[T, A](ys, function(y: T) {
        sum[A](map[T, A](xs, function(x: T) { h(x, y) }))
    }))
} by {
    define p(l: List[T]) -> Bool {
        sum[A](map[T, A](l, function(x: T) {
            sum[A](map[T, A](ys, function(y: T) { h(x, y) }))
        })) = sum[A](map[T, A](ys, function(y: T) {
            sum[A](map[T, A](l, function(x: T) { h(x, y) }))
        }))
    }
    forall(y: T) {
        map[T, A](List.nil[T], function(x: T) { h(x, y) }) = List.nil[A]
        sum[A](List.nil[A]) = A.0
        sum[A](map[T, A](List.nil[T], function(x: T) { h(x, y) })) = A.0
    }
    sum_map_zero_fn[T, A](ys, function(y: T) {
        sum[A](map[T, A](List.nil[T], function(x: T) { h(x, y) }))
    })
    sum[A](map[T, A](ys, function(y: T) {
        sum[A](map[T, A](List.nil[T], function(x: T) { h(x, y) }))
    })) = A.0
    map[T, A](List.nil[T], function(x: T) {
        sum[A](map[T, A](ys, function(y: T) { h(x, y) }))
    }) = List.nil[A]
    sum[A](List.nil[A]) = A.0
    sum[A](map[T, A](List.nil[T], function(x: T) {
        sum[A](map[T, A](ys, function(y: T) { h(x, y) }))
    })) = A.0
    p(List.nil[T])
    forall(x: T, xt: List[T]) {
        if p(xt) {
            p(xt) = (sum[A](map[T, A](xt, function(x2: T) {
                sum[A](map[T, A](ys, function(y: T) { h(x2, y) }))
            })) = sum[A](map[T, A](ys, function(y: T) {
                sum[A](map[T, A](xt, function(x2: T) { h(x2, y) }))
            })))
            sum_map_remove_one[T, A](List.cons(x, xt), x, function(x2: T) {
                sum[A](map[T, A](ys, function(y: T) { h(x2, y) }))
            })
            List.cons(x, xt).contains(x)
            sum[A](map[T, A](ys, function(y: T) { h(x, y) })) +
                sum[A](map[T, A](List.cons(x, xt).remove_one(x), function(x2: T) {
                    sum[A](map[T, A](ys, function(y: T) { h(x2, y) }))
                })) = sum[A](map[T, A](List.cons(x, xt), function(x2: T) {
                    sum[A](map[T, A](ys, function(y: T) { h(x2, y) }))
                }))
            remove_one_cons_eq(x, xt)
            List.cons(x, xt).remove_one(x) = xt
            sum[A](map[T, A](ys, function(y: T) { h(x, y) })) +
                sum[A](map[T, A](xt, function(x2: T) {
                    sum[A](map[T, A](ys, function(y: T) { h(x2, y) }))
                })) = sum[A](map[T, A](List.cons(x, xt), function(x2: T) {
                    sum[A](map[T, A](ys, function(y: T) { h(x2, y) }))
                }))
            sum[A](map[T, A](xt, function(x2: T) {
                sum[A](map[T, A](ys, function(y: T) { h(x2, y) }))
            })) = sum[A](map[T, A](ys, function(y: T) {
                sum[A](map[T, A](xt, function(x2: T) { h(x2, y) }))
            }))
            sum[A](map[T, A](List.cons(x, xt), function(x2: T) {
                sum[A](map[T, A](ys, function(y: T) { h(x2, y) }))
            })) = sum[A](map[T, A](ys, function(y: T) { h(x, y) })) +
                sum[A](map[T, A](ys, function(y: T) {
                    sum[A](map[T, A](xt, function(x2: T) { h(x2, y) }))
                }))
            sum_map_add_fn[T, A](ys, function(y: T) { h(x, y) }, function(y: T) {
                sum[A](map[T, A](xt, function(x2: T) { h(x2, y) }))
            })
            sum[A](map[T, A](ys, function(y: T) {
                h(x, y) + sum[A](map[T, A](xt, function(x2: T) { h(x2, y) }))
            })) = sum[A](map[T, A](ys, function(y: T) { h(x, y) })) +
                sum[A](map[T, A](ys, function(y: T) {
                    sum[A](map[T, A](xt, function(x2: T) { h(x2, y) }))
                }))
            forall(y: T) {
                sum_map_remove_one[T, A](List.cons(x, xt), x, function(x2: T) { h(x2, y) })
                List.cons(x, xt).contains(x)
                h(x, y) + sum[A](map[T, A](List.cons(x, xt).remove_one(x), function(x2: T) { h(x2, y) })) =
                    sum[A](map[T, A](List.cons(x, xt), function(x2: T) { h(x2, y) }))
                remove_one_cons_eq(x, xt)
                List.cons(x, xt).remove_one(x) = xt
                h(x, y) + sum[A](map[T, A](xt, function(x2: T) { h(x2, y) })) =
                    sum[A](map[T, A](List.cons(x, xt), function(x2: T) { h(x2, y) }))
            }
            sum_map_of_pointwise[T, A](ys, function(y: T) {
                sum[A](map[T, A](List.cons(x, xt), function(x2: T) { h(x2, y) }))
            }, function(y: T) {
                h(x, y) + sum[A](map[T, A](xt, function(x2: T) { h(x2, y) }))
            })
            sum[A](map[T, A](ys, function(y: T) {
                sum[A](map[T, A](List.cons(x, xt), function(x2: T) { h(x2, y) }))
            })) = sum[A](map[T, A](ys, function(y: T) {
                h(x, y) + sum[A](map[T, A](xt, function(x2: T) { h(x2, y) }))
            }))
            sum[A](map[T, A](List.cons(x, xt), function(x2: T) {
                sum[A](map[T, A](ys, function(y: T) { h(x2, y) }))
            })) = sum[A](map[T, A](ys, function(y: T) {
                sum[A](map[T, A](List.cons(x, xt), function(x2: T) { h(x2, y) }))
            }))
            p(List.cons(x, xt))
        }
    }
    p(List.nil[T]) and forall(head: T, tail: List[T]) { p(tail) implies p(List.cons(head, tail)) }
    List.induction(p)
    forall(l: List[T]) { p(l) }
    p(xs)
    sum[A](map[T, A](xs, function(x: T) {
        sum[A](map[T, A](ys, function(y: T) { h(x, y) }))
    })) = sum[A](map[T, A](ys, function(y: T) {
        sum[A](map[T, A](xs, function(x: T) { h(x, y) }))
    }))
}


/// A unique list does not contain its own head in its tail.
theorem unique_cons_not_contains_head[T](head: T, tail: List[T]) {
    List.cons(head, tail).is_unique implies not tail.contains(head)
} by {
    if List.cons(head, tail).is_unique {
        remove_one_unique_not_contains_self[T](List.cons(head, tail), head)
        not List.cons(head, tail).remove_one(head).contains(head)
        remove_one_cons_eq(head, tail)
        List.cons(head, tail).remove_one(head) = tail
        not tail.contains(head)
    }
}

/// One summand of a determinant-list expansion.
define det_row_sum_term[R: Ring, I](entry: (I, I) -> R, row: I, tail_rows: List[I], cols: List[I], col: I) -> R {
    entry(row, col) * alternating_sign[R](list_index_or_zero[I](cols, col)) *
        determinant_list[R, I](entry, tail_rows, cols.remove_one(col))
}

/// The head-row expansion expressed with the summand helper.
theorem determinant_list_cons_rows_term[R: Ring, I](entry: (I, I) -> R, row: I, tail_rows: List[I], cols: List[I]) {
    determinant_list[R, I](entry, List.cons(row, tail_rows), cols) =
        sum[R](map[I, R](cols, det_row_sum_term[R, I](entry, row, tail_rows, cols)))
} by {
    determinant_list_cons_rows[R, I](entry, row, tail_rows, cols)
    sum_map_of_pointwise[I, R](cols, function(col: I) {
        entry(row, col) * alternating_sign[R](list_index_or_zero[I](cols, col)) *
            determinant_list[R, I](entry, tail_rows, cols.remove_one(col))
    }, det_row_sum_term[R, I](entry, row, tail_rows, cols))
    sum[R](map[I, R](cols, det_row_sum_term[R, I](entry, row, tail_rows, cols))) =
        sum[R](map[I, R](cols, function(col: I) {
            entry(row, col) * alternating_sign[R](list_index_or_zero[I](cols, col)) *
                determinant_list[R, I](entry, tail_rows, cols.remove_one(col))
        }))
    determinant_list[R, I](entry, List.cons(row, tail_rows), cols) =
        sum[R](map[I, R](cols, det_row_sum_term[R, I](entry, row, tail_rows, cols)))
}

/// Removing one occurrence from a list shortens it by one.
theorem remove_one_length_suc[T](list: List[T], item: T) {
    list.contains(item) implies list.remove_one(item).length.suc = list.length
} by {
    define p(l: List[T]) -> Bool {
        l.contains(item) implies l.remove_one(item).length.suc = l.length
    }
    p(List.nil[T])
    forall(head: T, tail: List[T]) {
        if p(tail) {
            if List.cons(head, tail).contains(item) {
                if head = item {
                    remove_one_cons_eq(head, tail)
                    List.cons(head, tail).remove_one(item).length.suc = List.cons(head, tail).length
                }
                if head != item {
                    tail.contains(item)
                    remove_one_cons_neq(head, tail, item)
                    List.cons(head, tail).remove_one(item).length.suc = tail.remove_one(item).length.suc.suc
                    tail.remove_one(item).length.suc.suc = tail.length.suc
                    List.cons(head, tail).remove_one(item).length.suc = List.cons(head, tail).length
                }
                List.cons(head, tail).remove_one(item).length.suc = List.cons(head, tail).length
            }
            if not List.cons(head, tail).contains(item) {
            }
            p(List.cons(head, tail))
        }
    }
    List.induction(function(l: List[T]) { p(l) })
    forall(l: List[T]) { p(l) }
    p(list)
    list.contains(item) implies list.remove_one(item).length.suc = list.length
}

/// A scalar distributes over a mapped sum.
theorem sum_map_scalar_mul[T, R: Ring](items: List[T], c: R, f: T -> R) {
    c * sum[R](map[T, R](items, f)) = sum[R](map[T, R](items, function(x: T) { c * f(x) }))
} by {
    define p(l: List[T]) -> Bool {
        c * sum[R](map[T, R](l, f)) = sum[R](map[T, R](l, function(x: T) { c * f(x) }))
    }
    map[T, R](List.nil[T], f) = List.nil[R]
    sum[R](List.nil[R]) = R.0
    c * R.0 = R.0
    map[T, R](List.nil[T], function(x: T) { c * f(x) }) = List.nil[R]
    sum[R](List.nil[R]) = R.0
    p(List.nil[T])
    forall(head: T, tail: List[T]) {
        if p(tail) {
            p(tail) = (c * sum[R](map[T, R](tail, f)) = sum[R](map[T, R](tail, function(x: T) { c * f(x) })))
            sum_map_remove_one[T, R](List.cons(head, tail), head, f)
            List.cons(head, tail).contains(head)
            f(head) + sum[R](map[T, R](List.cons(head, tail).remove_one(head), f)) =
                sum[R](map[T, R](List.cons(head, tail), f))
            remove_one_cons_eq(head, tail)
            List.cons(head, tail).remove_one(head) = tail
            f(head) + sum[R](map[T, R](tail, f)) = sum[R](map[T, R](List.cons(head, tail), f))
            c * (f(head) + sum[R](map[T, R](tail, f))) = c * f(head) + c * sum[R](map[T, R](tail, f))
            c * sum[R](map[T, R](List.cons(head, tail), f)) = c * f(head) + c * sum[R](map[T, R](tail, f))
            c * f(head) + c * sum[R](map[T, R](tail, f)) = c * f(head) + sum[R](map[T, R](tail, function(x: T) { c * f(x) }))
            sum_map_remove_one[T, R](List.cons(head, tail), head, function(x: T) { c * f(x) })
            c * f(head) + sum[R](map[T, R](List.cons(head, tail).remove_one(head), function(x: T) { c * f(x) })) =
                sum[R](map[T, R](List.cons(head, tail), function(x: T) { c * f(x) }))
            c * f(head) + sum[R](map[T, R](tail, function(x: T) { c * f(x) })) =
                sum[R](map[T, R](List.cons(head, tail), function(x: T) { c * f(x) }))
            c * sum[R](map[T, R](List.cons(head, tail), f)) =
                sum[R](map[T, R](List.cons(head, tail), function(x: T) { c * f(x) }))
            p(List.cons(head, tail))
        }
    }
    p(List.nil[T]) and forall(head: T, tail: List[T]) { p(tail) implies p(List.cons(head, tail)) }
    List.induction(p)
    forall(l: List[T]) { p(l) }
    p(items)
    c * sum[R](map[T, R](items, f)) = sum[R](map[T, R](items, function(x: T) { c * f(x) }))
}

/// A determinant-list is unchanged when its entry function is replaced by a
/// pointwise-equal one.
theorem determinant_list_pointwise[R: Ring, I](e1: (I, I) -> R, e2: (I, I) -> R, rows: List[I], cols: List[I]) {
    (forall(r: I, c: I) { e1(r, c) = e2(r, c) }) implies
        determinant_list[R, I](e1, rows, cols) = determinant_list[R, I](e2, rows, cols)
} by {
    if forall(r: I, c: I) { e1(r, c) = e2(r, c) } {
        define p(l: List[I]) -> Bool {
            forall(cols2: List[I]) {
                determinant_list[R, I](e1, l, cols2) = determinant_list[R, I](e2, l, cols2)
            }
        }
        determinant_list_nil_rows[R, I](e1, List.nil[I])
        determinant_list_nil_rows[R, I](e2, List.nil[I])
        determinant_list[R, I](e1, List.nil[I], List.nil[I]) = R.1
        determinant_list[R, I](e2, List.nil[I], List.nil[I]) = R.1
        forall(cols2: List[I]) {
            determinant_list[R, I](e1, List.nil[I], cols2) = determinant_list[R, I](e2, List.nil[I], cols2)
        }
        p(List.nil[I])
        forall(head: I, tail: List[I]) {
            if p(tail) {
                forall(cols2: List[I]) {
                    p(tail) = (forall(cols3: List[I]) {
                        determinant_list[R, I](e1, tail, cols3) = determinant_list[R, I](e2, tail, cols3)
                    })
                    determinant_list_cons_rows_term[R, I](e1, head, tail, cols2)
                    determinant_list_cons_rows_term[R, I](e2, head, tail, cols2)
                    determinant_list[R, I](e1, List.cons(head, tail), cols2) =
                        sum[R](map[I, R](cols2, det_row_sum_term[R, I](e1, head, tail, cols2)))
                    determinant_list[R, I](e2, List.cons(head, tail), cols2) =
                        sum[R](map[I, R](cols2, det_row_sum_term[R, I](e2, head, tail, cols2)))
                    forall(col: I) {
                        if cols2.contains(col) {
                            e1(head, col) = e2(head, col)
                            det_row_sum_term[R, I](e1, head, tail, cols2, col) =
                                e1(head, col) * alternating_sign[R](list_index_or_zero[I](cols2, col)) *
                                    determinant_list[R, I](e1, tail, cols2.remove_one(col))
                            det_row_sum_term[R, I](e2, head, tail, cols2, col) =
                                e2(head, col) * alternating_sign[R](list_index_or_zero[I](cols2, col)) *
                                    determinant_list[R, I](e2, tail, cols2.remove_one(col))
                            determinant_list[R, I](e1, tail, cols2.remove_one(col)) =
                                determinant_list[R, I](e2, tail, cols2.remove_one(col))
                            det_row_sum_term[R, I](e1, head, tail, cols2, col) =
                                det_row_sum_term[R, I](e2, head, tail, cols2, col)
                        }
                    }
                    sum_map_of_pointwise[I, R](cols2, det_row_sum_term[R, I](e1, head, tail, cols2),
                        det_row_sum_term[R, I](e2, head, tail, cols2))
                    sum[R](map[I, R](cols2, det_row_sum_term[R, I](e1, head, tail, cols2))) =
                        sum[R](map[I, R](cols2, det_row_sum_term[R, I](e2, head, tail, cols2)))
                    determinant_list[R, I](e1, List.cons(head, tail), cols2) =
                        determinant_list[R, I](e2, List.cons(head, tail), cols2)
                }
                p(List.cons(head, tail))
            }
        }
        p(List.nil[I]) and forall(head: I, tail: List[I]) { p(tail) implies p(List.cons(head, tail)) }
        List.induction(p)
        forall(l: List[I]) { p(l) }
        p(rows)
        forall(cols3: List[I]) {
            determinant_list[R, I](e1, rows, cols3) = determinant_list[R, I](e2, rows, cols3)
        }
        determinant_list[R, I](e1, rows, cols) = determinant_list[R, I](e2, rows, cols)
    }
}

/// The determinant-list expansion is invariant under exchanging the roles of
/// rows and columns: `det(entry, rows, cols) = det(flip_entry, cols, rows)`
/// for row and column lists of equal length.
theorem nat_suc_eq_cancel(a: Nat, b: Nat) {
    a.suc = b.suc implies a = b
} by {
    if a.suc = b.suc {
        a.suc <= b.suc
        lte_cancel_suc(a, b)
        a <= b
        b.suc <= a.suc
        lte_cancel_suc(b, a)
        b <= a
        lte_antisymm(a, b)
        a = b
    }
}

theorem prod_exchange_comm[R: CommRing](a: R, b: R, s1: R, s2: R, s3: R, s4: R) {
    s1 * s2 = s3 * s4 implies (a * s1) * (b * s2) = (b * s3) * (a * s4)
} by {
    if s1 * s2 = s3 * s4 {
        (a * s1) * (b * s2) = a * s1 * (b * s2)
        a * s1 * (b * s2) = a * s1 * b * s2
        a * s1 * b * s2 = a * b * s1 * s2
        a * b * s1 * s2 = b * a * s1 * s2
        a * b * s1 * s2 = b * a * s3 * s4
        b * a * s3 * s4 = b * s3 * a * s4
        b * s3 * a * s4 = (b * s3) * (a * s4)
        (a * s1) * (b * s2) = (b * s3) * (a * s4)
    }
}
/// A predicate asserting the transpose identity for all row and column lists
/// of bounded length.
define det_transpose_pred[R: CommRing, I](entry: (I, I) -> R, n: Nat) -> Bool {
    forall(rows2: List[I], cols2: List[I]) {
        if rows2.length = cols2.length and rows2.length <= n and rows2.is_unique and cols2.is_unique {
            determinant_list[R, I](entry, rows2, cols2) =
                determinant_list[R, I](flip_entry[R, I](entry), cols2, rows2)
        }
    }
}

/// The strong-induction step for the transpose identity.
theorem det_transpose_step[R: CommRing, I](entry: (I, I) -> R, k: Nat) {
    true_below(det_transpose_pred[R, I](entry), k) implies
        forall(rows2: List[I], cols2: List[I]) {
            if rows2.length = cols2.length and rows2.length <= k and rows2.is_unique and cols2.is_unique {
                match rows2 {
                    List.nil {
                        determinant_list[R, I](entry, List.nil[I], cols2) =
                            determinant_list[R, I](flip_entry[R, I](entry), cols2, List.nil[I])
                    }
                    List.cons(head, tail) {
                        match cols2 {
                            List.nil {
                                determinant_list[R, I](entry, List.cons(head, tail), List.nil[I]) =
                                    determinant_list[R, I](flip_entry[R, I](entry), List.nil[I], List.cons(head, tail))
                            }
                            List.cons(c, cs) {
                                determinant_list[R, I](entry, List.cons(head, tail), List.cons(c, cs)) =
                                    determinant_list[R, I](flip_entry[R, I](entry), List.cons(c, cs), List.cons(head, tail))
                            }
                        }
                    }
                }
            }
        }
} by {
    if true_below(det_transpose_pred[R, I](entry), k) {
        forall(rows2: List[I], cols2: List[I]) {
            if rows2.length = cols2.length and rows2.length <= k and rows2.is_unique and cols2.is_unique {
                match rows2 {
                    List.nil {
                        match cols2 {
                            List.nil {
                                determinant_list_nil_rows[R, I](entry, List.nil[I])
                                determinant_list_nil_rows[R, I](flip_entry[R, I](entry), List.nil[I])
                                determinant_list[R, I](entry, List.nil[I], List.nil[I]) = R.1
                                determinant_list[R, I](flip_entry[R, I](entry), List.nil[I], List.nil[I]) = R.1
                                determinant_list[R, I](entry, List.nil[I], cols2) =
                                    determinant_list[R, I](flip_entry[R, I](entry), cols2, List.nil[I])
                            }
                            List.cons(c2, cs2) {
                                cols2.length = cs2.length.suc
                                rows2.length = Nat.0
                                Nat.0 = cs2.length.suc
                                false
                            }
                        }
                    }
                    List.cons(head, tail) {
                        match cols2 {
                            List.nil {
                                rows2.length = tail.length.suc
                                cols2.length = Nat.0
                                tail.length.suc = Nat.0
                                false
                            }
                            List.cons(c, cs) {
                                rows2.length = tail.length.suc
                                cols2.length = cs.length.suc
                                tail.length = cs.length
                                tail.length.suc <= k
                                match k {
                                    Nat.zero {
                                        rows2.length <= Nat.0
                                        tail.length.suc <= Nat.0
                                        Nat.0 < tail.length.suc
                                        lt_and_lte(Nat.0, tail.length.suc, Nat.0)
                                        Nat.0 < Nat.0
                                        lt_not_ref(Nat.0)
                                        false
                                    }
                                    Nat.suc(k2) {
                                        lte_cancel_suc(tail.length, k2)
                                        tail.length <= k2
                                        lt_suc(k2)
                                        k2 < k2.suc
                                        lte_and_lt(tail.length, k2, k2.suc)
                                        tail.length < Nat.suc(k2)
                                        tail.length < k
                                        unique_implies_tail_unique[I](head, tail)
                                        tail.is_unique
                                        unique_implies_tail_unique[I](c, cs)
                                        cs.is_unique
                                        unique_cons_not_contains_head[I](head, tail)
                                        not tail.contains(head)
                                        unique_cons_not_contains_head[I](c, cs)
                                        not cs.contains(c)
                                        true_below_apply(det_transpose_pred[R, I](entry), k, tail.length)
                                        det_transpose_pred[R, I](entry, tail.length)
                                        det_transpose_pred[R, I](entry, tail.length) = (forall(rows3: List[I], cols3: List[I]) {
                                            rows3.length = cols3.length and rows3.length <= tail.length and rows3.is_unique and cols3.is_unique implies
                                                determinant_list[R, I](entry, rows3, cols3) =
                                                    determinant_list[R, I](flip_entry[R, I](entry), cols3, rows3)
                                        })
                                        // LHS expansion along the head row
                                        determinant_list_cons_rows_term[R, I](entry, head, tail, List.cons(c, cs))
                                        determinant_list[R, I](entry, List.cons(head, tail), List.cons(c, cs)) =
                                            sum[R](map[I, R](List.cons(c, cs), det_row_sum_term[R, I](entry, head, tail, List.cons(c, cs))))
                                        sum_map_remove_one[I, R](List.cons(c, cs), c, det_row_sum_term[R, I](entry, head, tail, List.cons(c, cs)))
                                        List.cons(c, cs).contains(c)
                                        det_row_sum_term[R, I](entry, head, tail, List.cons(c, cs), c) +
                                            sum[R](map[I, R](List.cons(c, cs).remove_one(c), det_row_sum_term[R, I](entry, head, tail, List.cons(c, cs)))) =
                                            sum[R](map[I, R](List.cons(c, cs), det_row_sum_term[R, I](entry, head, tail, List.cons(c, cs))))
                                        remove_one_cons_eq[I](c, cs)
                                        List.cons(c, cs).remove_one(c) = cs
                                        det_row_sum_term[R, I](entry, head, tail, List.cons(c, cs), c) +
                                            sum[R](map[I, R](cs, det_row_sum_term[R, I](entry, head, tail, List.cons(c, cs)))) =
                                            sum[R](map[I, R](List.cons(c, cs), det_row_sum_term[R, I](entry, head, tail, List.cons(c, cs))))
                                        determinant_list[R, I](entry, List.cons(head, tail), List.cons(c, cs)) =
                                            det_row_sum_term[R, I](entry, head, tail, List.cons(c, cs), c) +
                                                sum[R](map[I, R](cs, det_row_sum_term[R, I](entry, head, tail, List.cons(c, cs))))
                                        list_index_or_zero_cons_self[I](c, cs)
                                        list_index_or_zero[I](List.cons(c, cs), c) = Nat.0
                                        alternating_sign_zero[R]
                                        alternating_sign[R](Nat.0) = R.1
                                        det_row_sum_term[R, I](entry, head, tail, List.cons(c, cs), c) =
                                            entry(head, c) * alternating_sign[R](list_index_or_zero[I](List.cons(c, cs), c)) *
                                                determinant_list[R, I](entry, tail, List.cons(c, cs).remove_one(c))
                                        det_row_sum_term[R, I](entry, head, tail, List.cons(c, cs), c) =
                                            entry(head, c) * R.1 * determinant_list[R, I](entry, tail, cs)
                                        entry(head, c) * R.1 = entry(head, c)
                                        det_row_sum_term[R, I](entry, head, tail, List.cons(c, cs), c) =
                                            entry(head, c) * determinant_list[R, I](entry, tail, cs)
                                        tail.length <= tail.length
                                        tail.is_unique
                                        cs.is_unique
                                        determinant_list[R, I](entry, tail, cs) =
                                            determinant_list[R, I](flip_entry[R, I](entry), cs, tail)
                                        det_row_sum_term[R, I](entry, head, tail, List.cons(c, cs), c) =
                                            entry(head, c) * determinant_list[R, I](flip_entry[R, I](entry), cs, tail)
                                        forall(x: I) {
                                            if cs.contains(x) {
                                                if x = c {
                                                    cs.contains(x) = cs.contains(c)
                                                    cs.contains(c) = false
                                                    cs.contains(x) = false
                                                    false
                                                }
                                                x != c
                                                list_index_or_zero_cons_other[I](c, cs, x)
                                                list_index_or_zero[I](List.cons(c, cs), x) =
                                                    list_index_or_zero[I](cs, x).suc
                                                remove_one_cons_neq[I](c, cs, x)
                                                List.cons(c, cs).remove_one(x) = List.cons(c, cs.remove_one(x))
                                                remove_one_length_suc[I](cs, x)
                                                cs.remove_one(x).length.suc = cs.length
                                                List.cons(c, cs).remove_one(x).length = cs.length
                                                tail.length = List.cons(c, cs).remove_one(x).length
                                                tail.length <= tail.length
                                                tail.is_unique
                                                remove_one_unique[I](List.cons(c, cs), x)
                                                List.cons(c, cs).is_unique
                                                List.cons(c, cs).remove_one(x).is_unique
                                                determinant_list[R, I](entry, tail, List.cons(c, cs).remove_one(x)) =
                                                    determinant_list[R, I](flip_entry[R, I](entry), List.cons(c, cs).remove_one(x), tail)
                                                det_row_sum_term[R, I](entry, head, tail, List.cons(c, cs), x) =
                                                    entry(head, x) * alternating_sign[R](list_index_or_zero[I](cs, x).suc) *
                                                        determinant_list[R, I](flip_entry[R, I](entry), List.cons(c, cs).remove_one(x), tail)
                                            }
                                        }
                                        sum_map_of_pointwise[I, R](cs, det_row_sum_term[R, I](entry, head, tail, List.cons(c, cs)),
                                            function(x: I) {
                                                entry(head, x) * alternating_sign[R](list_index_or_zero[I](cs, x).suc) *
                                                    determinant_list[R, I](flip_entry[R, I](entry), List.cons(c, cs).remove_one(x), tail)
                                            })
                                        sum[R](map[I, R](cs, det_row_sum_term[R, I](entry, head, tail, List.cons(c, cs)))) =
                                            sum[R](map[I, R](cs, function(x: I) {
                                                entry(head, x) * alternating_sign[R](list_index_or_zero[I](cs, x).suc) *
                                                    determinant_list[R, I](flip_entry[R, I](entry), List.cons(c, cs).remove_one(x), tail)
                                            }))
                                        determinant_list[R, I](entry, List.cons(head, tail), List.cons(c, cs)) =
                                            det_row_sum_term[R, I](entry, head, tail, List.cons(c, cs), c) +
                                                sum[R](map[I, R](cs, function(x: I) {
                                                    entry(head, x) * alternating_sign[R](list_index_or_zero[I](cs, x).suc) *
                                                        determinant_list[R, I](flip_entry[R, I](entry), List.cons(c, cs).remove_one(x), tail)
                                                }))
                                        forall(x: I) {
                                            if cs.contains(x) {
                                                if x = c {
                                                    cs.contains(x) = cs.contains(c)
                                                    cs.contains(c) = false
                                                    cs.contains(x) = false
                                                    false
                                                }
                                                x != c
                                                remove_one_cons_neq[I](c, cs, x)
                                                List.cons(c, cs).remove_one(x) = List.cons(c, cs.remove_one(x))
                                                entry(head, x) * alternating_sign[R](list_index_or_zero[I](cs, x).suc) *
                                                    determinant_list[R, I](flip_entry[R, I](entry), List.cons(c, cs).remove_one(x), tail) =
                                                    entry(head, x) * alternating_sign[R](list_index_or_zero[I](cs, x).suc) *
                                                        determinant_list[R, I](flip_entry[R, I](entry), List.cons(c, cs.remove_one(x)), tail)
                                            }
                                        }
                                        sum_map_of_pointwise[I, R](cs, function(x: I) {
                                            entry(head, x) * alternating_sign[R](list_index_or_zero[I](cs, x).suc) *
                                                determinant_list[R, I](flip_entry[R, I](entry), List.cons(c, cs).remove_one(x), tail)
                                        }, function(x: I) {
                                            entry(head, x) * alternating_sign[R](list_index_or_zero[I](cs, x).suc) *
                                                determinant_list[R, I](flip_entry[R, I](entry), List.cons(c, cs.remove_one(x)), tail)
                                        })
                                        sum[R](map[I, R](cs, function(x: I) {
                                            entry(head, x) * alternating_sign[R](list_index_or_zero[I](cs, x).suc) *
                                                determinant_list[R, I](flip_entry[R, I](entry), List.cons(c, cs).remove_one(x), tail)
                                        })) = sum[R](map[I, R](cs, function(x: I) {
                                            entry(head, x) * alternating_sign[R](list_index_or_zero[I](cs, x).suc) *
                                                determinant_list[R, I](flip_entry[R, I](entry), List.cons(c, cs.remove_one(x)), tail)
                                        }))
                                        determinant_list[R, I](entry, List.cons(head, tail), List.cons(c, cs)) =
                                            det_row_sum_term[R, I](entry, head, tail, List.cons(c, cs), c) +
                                                sum[R](map[I, R](cs, function(x: I) {
                                                    entry(head, x) * alternating_sign[R](list_index_or_zero[I](cs, x).suc) *
                                                        determinant_list[R, I](flip_entry[R, I](entry), List.cons(c, cs.remove_one(x)), tail)
                                                }))
                                        forall(x: I) {
                                            if cs.contains(x) {
                                                determinant_list_cons_rows_term[R, I](flip_entry[R, I](entry), c, cs.remove_one(x), tail)
                                                determinant_list[R, I](flip_entry[R, I](entry), List.cons(c, cs.remove_one(x)), tail) =
                                                    sum[R](map[I, R](tail, det_row_sum_term[R, I](flip_entry[R, I](entry), c, cs.remove_one(x), tail)))
                                                entry(head, x) * alternating_sign[R](list_index_or_zero[I](cs, x).suc) *
                                                    determinant_list[R, I](flip_entry[R, I](entry), List.cons(c, cs.remove_one(x)), tail) =
                                                    entry(head, x) * alternating_sign[R](list_index_or_zero[I](cs, x).suc) *
                                                        sum[R](map[I, R](tail, det_row_sum_term[R, I](flip_entry[R, I](entry), c, cs.remove_one(x), tail)))
                                            }
                                        }
                                        sum_map_of_pointwise[I, R](cs, function(x: I) {
                                            entry(head, x) * alternating_sign[R](list_index_or_zero[I](cs, x).suc) *
                                                determinant_list[R, I](flip_entry[R, I](entry), List.cons(c, cs.remove_one(x)), tail)
                                        }, function(x: I) {
                                            entry(head, x) * alternating_sign[R](list_index_or_zero[I](cs, x).suc) *
                                                sum[R](map[I, R](tail, det_row_sum_term[R, I](flip_entry[R, I](entry), c, cs.remove_one(x), tail)))
                                        })
                                        sum[R](map[I, R](cs, function(x: I) {
                                            entry(head, x) * alternating_sign[R](list_index_or_zero[I](cs, x).suc) *
                                                determinant_list[R, I](flip_entry[R, I](entry), List.cons(c, cs.remove_one(x)), tail)
                                        })) = sum[R](map[I, R](cs, function(x: I) {
                                            entry(head, x) * alternating_sign[R](list_index_or_zero[I](cs, x).suc) *
                                                sum[R](map[I, R](tail, det_row_sum_term[R, I](flip_entry[R, I](entry), c, cs.remove_one(x), tail)))
                                        }))
                                        determinant_list[R, I](entry, List.cons(head, tail), List.cons(c, cs)) =
                                            det_row_sum_term[R, I](entry, head, tail, List.cons(c, cs), c) +
                                                sum[R](map[I, R](cs, function(x: I) {
                                                    entry(head, x) * alternating_sign[R](list_index_or_zero[I](cs, x).suc) *
                                                        sum[R](map[I, R](tail, det_row_sum_term[R, I](flip_entry[R, I](entry), c, cs.remove_one(x), tail)))
                                                }))
                                        forall(x: I) {
                                            if cs.contains(x) {
                                                sum_map_scalar_mul[I, R](tail,
                                                    entry(head, x) * alternating_sign[R](list_index_or_zero[I](cs, x).suc),
                                                    det_row_sum_term[R, I](flip_entry[R, I](entry), c, cs.remove_one(x), tail))
                                                (entry(head, x) * alternating_sign[R](list_index_or_zero[I](cs, x).suc)) *
                                                    sum[R](map[I, R](tail, det_row_sum_term[R, I](flip_entry[R, I](entry), c, cs.remove_one(x), tail))) =
                                                    sum[R](map[I, R](tail, function(y: I) {
                                                        (entry(head, x) * alternating_sign[R](list_index_or_zero[I](cs, x).suc)) *
                                                            det_row_sum_term[R, I](flip_entry[R, I](entry), c, cs.remove_one(x), tail, y)
                                                    }))
                                                entry(head, x) * alternating_sign[R](list_index_or_zero[I](cs, x).suc) *
                                                    sum[R](map[I, R](tail, det_row_sum_term[R, I](flip_entry[R, I](entry), c, cs.remove_one(x), tail))) =
                                                    sum[R](map[I, R](tail, function(y: I) {
                                                        (entry(head, x) * alternating_sign[R](list_index_or_zero[I](cs, x).suc)) *
                                                            det_row_sum_term[R, I](flip_entry[R, I](entry), c, cs.remove_one(x), tail, y)
                                                    }))
                                            }
                                        }
                                        sum_map_of_pointwise[I, R](cs, function(x: I) {
                                            entry(head, x) * alternating_sign[R](list_index_or_zero[I](cs, x).suc) *
                                                sum[R](map[I, R](tail, det_row_sum_term[R, I](flip_entry[R, I](entry), c, cs.remove_one(x), tail)))
                                        }, function(x: I) {
                                            sum[R](map[I, R](tail, function(y: I) {
                                                (entry(head, x) * alternating_sign[R](list_index_or_zero[I](cs, x).suc)) *
                                                    det_row_sum_term[R, I](flip_entry[R, I](entry), c, cs.remove_one(x), tail, y)
                                            }))
                                        })
                                        sum[R](map[I, R](cs, function(x: I) {
                                            entry(head, x) * alternating_sign[R](list_index_or_zero[I](cs, x).suc) *
                                                sum[R](map[I, R](tail, det_row_sum_term[R, I](flip_entry[R, I](entry), c, cs.remove_one(x), tail)))
                                        })) = sum[R](map[I, R](cs, function(x: I) {
                                            sum[R](map[I, R](tail, function(y: I) {
                                                (entry(head, x) * alternating_sign[R](list_index_or_zero[I](cs, x).suc)) *
                                                    det_row_sum_term[R, I](flip_entry[R, I](entry), c, cs.remove_one(x), tail, y)
                                            }))
                                        }))
                                        determinant_list[R, I](entry, List.cons(head, tail), List.cons(c, cs)) =
                                            det_row_sum_term[R, I](entry, head, tail, List.cons(c, cs), c) +
                                                sum[R](map[I, R](cs, function(x: I) {
                                                    sum[R](map[I, R](tail, function(y: I) {
                                                        (entry(head, x) * alternating_sign[R](list_index_or_zero[I](cs, x).suc)) *
                                                            det_row_sum_term[R, I](flip_entry[R, I](entry), c, cs.remove_one(x), tail, y)
                                                    }))
                                                }))
                                        // rewrite the inner y-terms and apply the length-(n-2) induction hypothesis
                                        forall(x: I) {
                                            if cs.contains(x) {
                                                forall(y: I) {
                                                    if tail.contains(y) {
                                                        if y = head {
                                                            tail.contains(y) = tail.contains(head)
                                                            tail.contains(head) = false
                                                            tail.contains(y) = false
                                                            false
                                                        }
                                                        y != head
                                                        remove_one_length_suc[I](tail, y)
                                                        tail.remove_one(y).length.suc = tail.length
                                                        remove_one_length_suc[I](cs, x)
                                                        cs.remove_one(x).length.suc = cs.length
                                                        tail.length = cs.length
                                                        tail.remove_one(y).length = cs.remove_one(x).length
                                                        tail.remove_one(y).length <= tail.length
                                                        remove_one_unique[I](tail, y)
                                                        tail.is_unique
                                                        tail.remove_one(y).is_unique
                                                        remove_one_unique[I](cs, x)
                                                        cs.remove_one(x).is_unique
                                                        determinant_list[R, I](flip_entry[R, I](entry), cs.remove_one(x), tail.remove_one(y)) =
                                                            determinant_list[R, I](entry, tail.remove_one(y), cs.remove_one(x))
                                                        det_row_sum_term[R, I](flip_entry[R, I](entry), c, cs.remove_one(x), tail, y) =
                                                            flip_entry[R, I](entry, c, y) *
                                                                alternating_sign[R](list_index_or_zero[I](tail, y)) *
                                                                determinant_list[R, I](flip_entry[R, I](entry), cs.remove_one(x), tail.remove_one(y))
                                                        flip_entry[R, I](entry, c, y) = entry(y, c)
                                                        det_row_sum_term[R, I](flip_entry[R, I](entry), c, cs.remove_one(x), tail, y) =
                                                            entry(y, c) * alternating_sign[R](list_index_or_zero[I](tail, y)) *
                                                                determinant_list[R, I](entry, tail.remove_one(y), cs.remove_one(x))
                                                        (entry(head, x) * alternating_sign[R](list_index_or_zero[I](cs, x).suc)) *
                                                            det_row_sum_term[R, I](flip_entry[R, I](entry), c, cs.remove_one(x), tail, y) =
                                                            (entry(head, x) * alternating_sign[R](list_index_or_zero[I](cs, x).suc)) *
                                                                entry(y, c) * alternating_sign[R](list_index_or_zero[I](tail, y)) *
                                                                determinant_list[R, I](entry, tail.remove_one(y), cs.remove_one(x))
                                                    }
                                                }
                                                sum_map_of_pointwise[I, R](tail, function(y: I) {
                                                    (entry(head, x) * alternating_sign[R](list_index_or_zero[I](cs, x).suc)) *
                                                        det_row_sum_term[R, I](flip_entry[R, I](entry), c, cs.remove_one(x), tail, y)
                                                }, function(y: I) {
                                                    (entry(head, x) * alternating_sign[R](list_index_or_zero[I](cs, x).suc)) *
                                                        entry(y, c) * alternating_sign[R](list_index_or_zero[I](tail, y)) *
                                                        determinant_list[R, I](entry, tail.remove_one(y), cs.remove_one(x))
                                                })
                                                sum[R](map[I, R](tail, function(y: I) {
                                                    (entry(head, x) * alternating_sign[R](list_index_or_zero[I](cs, x).suc)) *
                                                        det_row_sum_term[R, I](flip_entry[R, I](entry), c, cs.remove_one(x), tail, y)
                                                })) = sum[R](map[I, R](tail, function(y: I) {
                                                    (entry(head, x) * alternating_sign[R](list_index_or_zero[I](cs, x).suc)) *
                                                        entry(y, c) * alternating_sign[R](list_index_or_zero[I](tail, y)) *
                                                        determinant_list[R, I](entry, tail.remove_one(y), cs.remove_one(x))
                                                }))
                                            }
                                        }
                                        sum_map_of_pointwise[I, R](cs, function(x: I) {
                                            sum[R](map[I, R](tail, function(y: I) {
                                                (entry(head, x) * alternating_sign[R](list_index_or_zero[I](cs, x).suc)) *
                                                    det_row_sum_term[R, I](flip_entry[R, I](entry), c, cs.remove_one(x), tail, y)
                                            }))
                                        }, function(x: I) {
                                            sum[R](map[I, R](tail, function(y: I) {
                                                (entry(head, x) * alternating_sign[R](list_index_or_zero[I](cs, x).suc)) *
                                                    entry(y, c) * alternating_sign[R](list_index_or_zero[I](tail, y)) *
                                                    determinant_list[R, I](entry, tail.remove_one(y), cs.remove_one(x))
                                            }))
                                        })
                                        sum[R](map[I, R](cs, function(x: I) {
                                            sum[R](map[I, R](tail, function(y: I) {
                                                (entry(head, x) * alternating_sign[R](list_index_or_zero[I](cs, x).suc)) *
                                                    det_row_sum_term[R, I](flip_entry[R, I](entry), c, cs.remove_one(x), tail, y)
                                            }))
                                        })) = sum[R](map[I, R](cs, function(x: I) {
                                            sum[R](map[I, R](tail, function(y: I) {
                                                (entry(head, x) * alternating_sign[R](list_index_or_zero[I](cs, x).suc)) *
                                                    entry(y, c) * alternating_sign[R](list_index_or_zero[I](tail, y)) *
                                                    determinant_list[R, I](entry, tail.remove_one(y), cs.remove_one(x))
                                            }))
                                        }))
                                        determinant_list[R, I](entry, List.cons(head, tail), List.cons(c, cs)) =
                                            det_row_sum_term[R, I](entry, head, tail, List.cons(c, cs), c) +
                                                sum[R](map[I, R](cs, function(x: I) {
                                                    sum[R](map[I, R](tail, function(y: I) {
                                                        (entry(head, x) * alternating_sign[R](list_index_or_zero[I](cs, x).suc)) *
                                                            entry(y, c) * alternating_sign[R](list_index_or_zero[I](tail, y)) *
                                                            determinant_list[R, I](entry, tail.remove_one(y), cs.remove_one(x))
                                                    }))
                                                }))
                                        // RHS expansion along the first column
                                        determinant_list_cons_rows_term[R, I](flip_entry[R, I](entry), c, cs, List.cons(head, tail))
                                        determinant_list[R, I](flip_entry[R, I](entry), List.cons(c, cs), List.cons(head, tail)) =
                                            sum[R](map[I, R](List.cons(head, tail), det_row_sum_term[R, I](flip_entry[R, I](entry), c, cs, List.cons(head, tail))))
                                        sum_map_remove_one[I, R](List.cons(head, tail), head, det_row_sum_term[R, I](flip_entry[R, I](entry), c, cs, List.cons(head, tail)))
                                        List.cons(head, tail).contains(head)
                                        det_row_sum_term[R, I](flip_entry[R, I](entry), c, cs, List.cons(head, tail), head) +
                                            sum[R](map[I, R](List.cons(head, tail).remove_one(head), det_row_sum_term[R, I](flip_entry[R, I](entry), c, cs, List.cons(head, tail)))) =
                                            sum[R](map[I, R](List.cons(head, tail), det_row_sum_term[R, I](flip_entry[R, I](entry), c, cs, List.cons(head, tail))))
                                        remove_one_cons_eq[I](head, tail)
                                        List.cons(head, tail).remove_one(head) = tail
                                        det_row_sum_term[R, I](flip_entry[R, I](entry), c, cs, List.cons(head, tail), head) +
                                            sum[R](map[I, R](tail, det_row_sum_term[R, I](flip_entry[R, I](entry), c, cs, List.cons(head, tail)))) =
                                            sum[R](map[I, R](List.cons(head, tail), det_row_sum_term[R, I](flip_entry[R, I](entry), c, cs, List.cons(head, tail))))
                                        determinant_list[R, I](flip_entry[R, I](entry), List.cons(c, cs), List.cons(head, tail)) =
                                            det_row_sum_term[R, I](flip_entry[R, I](entry), c, cs, List.cons(head, tail), head) +
                                                sum[R](map[I, R](tail, det_row_sum_term[R, I](flip_entry[R, I](entry), c, cs, List.cons(head, tail))))
                                        list_index_or_zero_cons_self[I](head, tail)
                                        list_index_or_zero[I](List.cons(head, tail), head) = Nat.0
                                        alternating_sign_zero[R]
                                        alternating_sign[R](Nat.0) = R.1
                                        det_row_sum_term[R, I](flip_entry[R, I](entry), c, cs, List.cons(head, tail), head) =
                                            flip_entry[R, I](entry, c, head) *
                                                alternating_sign[R](list_index_or_zero[I](List.cons(head, tail), head)) *
                                                determinant_list[R, I](flip_entry[R, I](entry), cs, List.cons(head, tail).remove_one(head))
                                        flip_entry[R, I](entry, c, head) = entry(head, c)
                                        det_row_sum_term[R, I](flip_entry[R, I](entry), c, cs, List.cons(head, tail), head) =
                                            entry(head, c) * R.1 * determinant_list[R, I](flip_entry[R, I](entry), cs, tail)
                                        entry(head, c) * R.1 = entry(head, c)
                                        det_row_sum_term[R, I](flip_entry[R, I](entry), c, cs, List.cons(head, tail), head) =
                                            entry(head, c) * determinant_list[R, I](flip_entry[R, I](entry), cs, tail)
                                        forall(y: I) {
                                            if tail.contains(y) {
                                                if y = head {
                                                    tail.contains(y) = tail.contains(head)
                                                    tail.contains(head) = false
                                                    tail.contains(y) = false
                                                    false
                                                }
                                                y != head
                                                list_index_or_zero_cons_other[I](head, tail, y)
                                                list_index_or_zero[I](List.cons(head, tail), y) =
                                                    list_index_or_zero[I](tail, y).suc
                                                remove_one_cons_neq[I](head, tail, y)
                                                List.cons(head, tail).remove_one(y) = List.cons(head, tail.remove_one(y))
                                                remove_one_length_suc[I](tail, y)
                                                tail.remove_one(y).length.suc = tail.length
                                                List.cons(head, tail).remove_one(y).length = tail.length
                                                List.cons(head, tail).remove_one(y).length = cs.length
                                                List.cons(head, tail).remove_one(y).length <= tail.length
                                                remove_one_unique[I](List.cons(head, tail), y)
                                                List.cons(head, tail).is_unique
                                                List.cons(head, tail).remove_one(y).is_unique
                                                cs.is_unique
                                                determinant_list[R, I](flip_entry[R, I](entry), cs, List.cons(head, tail).remove_one(y)) =
                                                    determinant_list[R, I](entry, List.cons(head, tail).remove_one(y), cs)
                                                det_row_sum_term[R, I](flip_entry[R, I](entry), c, cs, List.cons(head, tail), y) =
                                                    flip_entry[R, I](entry, c, y) *
                                                        alternating_sign[R](list_index_or_zero[I](List.cons(head, tail), y)) *
                                                        determinant_list[R, I](flip_entry[R, I](entry), cs, List.cons(head, tail).remove_one(y))
                                                flip_entry[R, I](entry, c, y) = entry(y, c)
                                                det_row_sum_term[R, I](flip_entry[R, I](entry), c, cs, List.cons(head, tail), y) =
                                                    entry(y, c) * alternating_sign[R](list_index_or_zero[I](tail, y).suc) *
                                                        determinant_list[R, I](entry, List.cons(head, tail).remove_one(y), cs)
                                            }
                                        }
                                        sum_map_of_pointwise[I, R](tail, det_row_sum_term[R, I](flip_entry[R, I](entry), c, cs, List.cons(head, tail)),
                                            function(y: I) {
                                                entry(y, c) * alternating_sign[R](list_index_or_zero[I](tail, y).suc) *
                                                    determinant_list[R, I](entry, List.cons(head, tail).remove_one(y), cs)
                                            })
                                        sum[R](map[I, R](tail, det_row_sum_term[R, I](flip_entry[R, I](entry), c, cs, List.cons(head, tail)))) =
                                            sum[R](map[I, R](tail, function(y: I) {
                                                entry(y, c) * alternating_sign[R](list_index_or_zero[I](tail, y).suc) *
                                                    determinant_list[R, I](entry, List.cons(head, tail).remove_one(y), cs)
                                            }))
                                        determinant_list[R, I](flip_entry[R, I](entry), List.cons(c, cs), List.cons(head, tail)) =
                                            det_row_sum_term[R, I](flip_entry[R, I](entry), c, cs, List.cons(head, tail), head) +
                                                sum[R](map[I, R](tail, function(y: I) {
                                                    entry(y, c) * alternating_sign[R](list_index_or_zero[I](tail, y).suc) *
                                                        determinant_list[R, I](entry, List.cons(head, tail).remove_one(y), cs)
                                                }))
                                        forall(y: I) {
                                            if tail.contains(y) {
                                                if y = head {
                                                    tail.contains(y) = tail.contains(head)
                                                    tail.contains(head) = false
                                                    tail.contains(y) = false
                                                    false
                                                }
                                                y != head
                                                remove_one_cons_neq[I](head, tail, y)
                                                List.cons(head, tail).remove_one(y) = List.cons(head, tail.remove_one(y))
                                                entry(y, c) * alternating_sign[R](list_index_or_zero[I](tail, y).suc) *
                                                    determinant_list[R, I](entry, List.cons(head, tail).remove_one(y), cs) =
                                                    entry(y, c) * alternating_sign[R](list_index_or_zero[I](tail, y).suc) *
                                                        determinant_list[R, I](entry, List.cons(head, tail.remove_one(y)), cs)
                                            }
                                        }
                                        sum_map_of_pointwise[I, R](tail, function(y: I) {
                                            entry(y, c) * alternating_sign[R](list_index_or_zero[I](tail, y).suc) *
                                                determinant_list[R, I](entry, List.cons(head, tail).remove_one(y), cs)
                                        }, function(y: I) {
                                            entry(y, c) * alternating_sign[R](list_index_or_zero[I](tail, y).suc) *
                                                determinant_list[R, I](entry, List.cons(head, tail.remove_one(y)), cs)
                                        })
                                        sum[R](map[I, R](tail, function(y: I) {
                                            entry(y, c) * alternating_sign[R](list_index_or_zero[I](tail, y).suc) *
                                                determinant_list[R, I](entry, List.cons(head, tail).remove_one(y), cs)
                                        })) = sum[R](map[I, R](tail, function(y: I) {
                                            entry(y, c) * alternating_sign[R](list_index_or_zero[I](tail, y).suc) *
                                                determinant_list[R, I](entry, List.cons(head, tail.remove_one(y)), cs)
                                        }))
                                        determinant_list[R, I](flip_entry[R, I](entry), List.cons(c, cs), List.cons(head, tail)) =
                                            det_row_sum_term[R, I](flip_entry[R, I](entry), c, cs, List.cons(head, tail), head) +
                                                sum[R](map[I, R](tail, function(y: I) {
                                                    entry(y, c) * alternating_sign[R](list_index_or_zero[I](tail, y).suc) *
                                                        determinant_list[R, I](entry, List.cons(head, tail.remove_one(y)), cs)
                                                }))
                                        forall(y: I) {
                                            if tail.contains(y) {
                                                determinant_list_cons_rows_term[R, I](entry, head, tail.remove_one(y), cs)
                                                determinant_list[R, I](entry, List.cons(head, tail.remove_one(y)), cs) =
                                                    sum[R](map[I, R](cs, det_row_sum_term[R, I](entry, head, tail.remove_one(y), cs)))
                                                entry(y, c) * alternating_sign[R](list_index_or_zero[I](tail, y).suc) *
                                                    determinant_list[R, I](entry, List.cons(head, tail.remove_one(y)), cs) =
                                                    entry(y, c) * alternating_sign[R](list_index_or_zero[I](tail, y).suc) *
                                                        sum[R](map[I, R](cs, det_row_sum_term[R, I](entry, head, tail.remove_one(y), cs)))
                                            }
                                        }
                                        sum_map_of_pointwise[I, R](tail, function(y: I) {
                                            entry(y, c) * alternating_sign[R](list_index_or_zero[I](tail, y).suc) *
                                                determinant_list[R, I](entry, List.cons(head, tail.remove_one(y)), cs)
                                        }, function(y: I) {
                                            entry(y, c) * alternating_sign[R](list_index_or_zero[I](tail, y).suc) *
                                                sum[R](map[I, R](cs, det_row_sum_term[R, I](entry, head, tail.remove_one(y), cs)))
                                        })
                                        sum[R](map[I, R](tail, function(y: I) {
                                            entry(y, c) * alternating_sign[R](list_index_or_zero[I](tail, y).suc) *
                                                determinant_list[R, I](entry, List.cons(head, tail.remove_one(y)), cs)
                                        })) = sum[R](map[I, R](tail, function(y: I) {
                                            entry(y, c) * alternating_sign[R](list_index_or_zero[I](tail, y).suc) *
                                                sum[R](map[I, R](cs, det_row_sum_term[R, I](entry, head, tail.remove_one(y), cs)))
                                        }))
                                        determinant_list[R, I](flip_entry[R, I](entry), List.cons(c, cs), List.cons(head, tail)) =
                                            det_row_sum_term[R, I](flip_entry[R, I](entry), c, cs, List.cons(head, tail), head) +
                                                sum[R](map[I, R](tail, function(y: I) {
                                                    entry(y, c) * alternating_sign[R](list_index_or_zero[I](tail, y).suc) *
                                                        sum[R](map[I, R](cs, det_row_sum_term[R, I](entry, head, tail.remove_one(y), cs)))
                                                }))
                                        forall(y: I) {
                                            if tail.contains(y) {
                                                sum_map_scalar_mul[I, R](cs,
                                                    entry(y, c) * alternating_sign[R](list_index_or_zero[I](tail, y).suc),
                                                    det_row_sum_term[R, I](entry, head, tail.remove_one(y), cs))
                                                (entry(y, c) * alternating_sign[R](list_index_or_zero[I](tail, y).suc)) *
                                                    sum[R](map[I, R](cs, det_row_sum_term[R, I](entry, head, tail.remove_one(y), cs))) =
                                                    sum[R](map[I, R](cs, function(x: I) {
                                                        (entry(y, c) * alternating_sign[R](list_index_or_zero[I](tail, y).suc)) *
                                                            det_row_sum_term[R, I](entry, head, tail.remove_one(y), cs, x)
                                                    }))
                                                entry(y, c) * alternating_sign[R](list_index_or_zero[I](tail, y).suc) *
                                                    sum[R](map[I, R](cs, det_row_sum_term[R, I](entry, head, tail.remove_one(y), cs))) =
                                                    sum[R](map[I, R](cs, function(x: I) {
                                                        (entry(y, c) * alternating_sign[R](list_index_or_zero[I](tail, y).suc)) *
                                                            det_row_sum_term[R, I](entry, head, tail.remove_one(y), cs, x)
                                                    }))
                                            }
                                        }
                                        sum_map_of_pointwise[I, R](tail, function(y: I) {
                                            entry(y, c) * alternating_sign[R](list_index_or_zero[I](tail, y).suc) *
                                                sum[R](map[I, R](cs, det_row_sum_term[R, I](entry, head, tail.remove_one(y), cs)))
                                        }, function(y: I) {
                                            sum[R](map[I, R](cs, function(x: I) {
                                                (entry(y, c) * alternating_sign[R](list_index_or_zero[I](tail, y).suc)) *
                                                    det_row_sum_term[R, I](entry, head, tail.remove_one(y), cs, x)
                                            }))
                                        })
                                        sum[R](map[I, R](tail, function(y: I) {
                                            entry(y, c) * alternating_sign[R](list_index_or_zero[I](tail, y).suc) *
                                                sum[R](map[I, R](cs, det_row_sum_term[R, I](entry, head, tail.remove_one(y), cs)))
                                        })) = sum[R](map[I, R](tail, function(y: I) {
                                            sum[R](map[I, R](cs, function(x: I) {
                                                (entry(y, c) * alternating_sign[R](list_index_or_zero[I](tail, y).suc)) *
                                                    det_row_sum_term[R, I](entry, head, tail.remove_one(y), cs, x)
                                            }))
                                        }))
                                        determinant_list[R, I](flip_entry[R, I](entry), List.cons(c, cs), List.cons(head, tail)) =
                                            det_row_sum_term[R, I](flip_entry[R, I](entry), c, cs, List.cons(head, tail), head) +
                                                sum[R](map[I, R](tail, function(y: I) {
                                                    sum[R](map[I, R](cs, function(x: I) {
                                                        (entry(y, c) * alternating_sign[R](list_index_or_zero[I](tail, y).suc)) *
                                                            det_row_sum_term[R, I](entry, head, tail.remove_one(y), cs, x)
                                                    }))
                                                }))
                                        forall(y: I) {
                                            if tail.contains(y) {
                                                forall(x: I) {
                                                    if cs.contains(x) {
                                                        remove_one_length_suc[I](cs, x)
                                                        cs.remove_one(x).length.suc = cs.length
                                                        remove_one_length_suc[I](tail, y)
                                                        tail.remove_one(y).length.suc = tail.length
                                                        tail.length = cs.length
                                                        tail.remove_one(y).length = cs.remove_one(x).length
                                                        tail.remove_one(y).length <= tail.length
                                                        remove_one_unique[I](tail, y)
                                                        tail.is_unique
                                                        tail.remove_one(y).is_unique
                                                        remove_one_unique[I](cs, x)
                                                        cs.remove_one(x).is_unique
                                                        determinant_list[R, I](entry, tail.remove_one(y), cs.remove_one(x)) =
                                                            determinant_list[R, I](flip_entry[R, I](entry), cs.remove_one(x), tail.remove_one(y))
                                                        det_row_sum_term[R, I](entry, head, tail.remove_one(y), cs, x) =
                                                            entry(head, x) * alternating_sign[R](list_index_or_zero[I](cs, x)) *
                                                                determinant_list[R, I](entry, tail.remove_one(y), cs.remove_one(x))
                                                        (entry(y, c) * alternating_sign[R](list_index_or_zero[I](tail, y).suc)) *
                                                            det_row_sum_term[R, I](entry, head, tail.remove_one(y), cs, x) =
                                                            (entry(y, c) * alternating_sign[R](list_index_or_zero[I](tail, y).suc)) *
                                                                entry(head, x) * alternating_sign[R](list_index_or_zero[I](cs, x)) *
                                                                determinant_list[R, I](entry, tail.remove_one(y), cs.remove_one(x))
                                                    }
                                                }
                                                sum_map_of_pointwise[I, R](cs, function(x: I) {
                                                    (entry(y, c) * alternating_sign[R](list_index_or_zero[I](tail, y).suc)) *
                                                        det_row_sum_term[R, I](entry, head, tail.remove_one(y), cs, x)
                                                }, function(x: I) {
                                                    (entry(y, c) * alternating_sign[R](list_index_or_zero[I](tail, y).suc)) *
                                                        entry(head, x) * alternating_sign[R](list_index_or_zero[I](cs, x)) *
                                                        determinant_list[R, I](entry, tail.remove_one(y), cs.remove_one(x))
                                                })
                                                sum[R](map[I, R](cs, function(x: I) {
                                                    (entry(y, c) * alternating_sign[R](list_index_or_zero[I](tail, y).suc)) *
                                                        det_row_sum_term[R, I](entry, head, tail.remove_one(y), cs, x)
                                                })) = sum[R](map[I, R](cs, function(x: I) {
                                                    (entry(y, c) * alternating_sign[R](list_index_or_zero[I](tail, y).suc)) *
                                                        entry(head, x) * alternating_sign[R](list_index_or_zero[I](cs, x)) *
                                                        determinant_list[R, I](entry, tail.remove_one(y), cs.remove_one(x))
                                                }))
                                            }
                                        }
                                        sum_map_of_pointwise[I, R](tail, function(y: I) {
                                            sum[R](map[I, R](cs, function(x: I) {
                                                (entry(y, c) * alternating_sign[R](list_index_or_zero[I](tail, y).suc)) *
                                                    det_row_sum_term[R, I](entry, head, tail.remove_one(y), cs, x)
                                            }))
                                        }, function(y: I) {
                                            sum[R](map[I, R](cs, function(x: I) {
                                                (entry(y, c) * alternating_sign[R](list_index_or_zero[I](tail, y).suc)) *
                                                    entry(head, x) * alternating_sign[R](list_index_or_zero[I](cs, x)) *
                                                    determinant_list[R, I](entry, tail.remove_one(y), cs.remove_one(x))
                                            }))
                                        })
                                        sum[R](map[I, R](tail, function(y: I) {
                                            sum[R](map[I, R](cs, function(x: I) {
                                                (entry(y, c) * alternating_sign[R](list_index_or_zero[I](tail, y).suc)) *
                                                    det_row_sum_term[R, I](entry, head, tail.remove_one(y), cs, x)
                                            }))
                                        })) = sum[R](map[I, R](tail, function(y: I) {
                                            sum[R](map[I, R](cs, function(x: I) {
                                                (entry(y, c) * alternating_sign[R](list_index_or_zero[I](tail, y).suc)) *
                                                    entry(head, x) * alternating_sign[R](list_index_or_zero[I](cs, x)) *
                                                    determinant_list[R, I](entry, tail.remove_one(y), cs.remove_one(x))
                                            }))
                                        }))
                                        determinant_list[R, I](flip_entry[R, I](entry), List.cons(c, cs), List.cons(head, tail)) =
                                            det_row_sum_term[R, I](flip_entry[R, I](entry), c, cs, List.cons(head, tail), head) +
                                                sum[R](map[I, R](tail, function(y: I) {
                                                    sum[R](map[I, R](cs, function(x: I) {
                                                        (entry(y, c) * alternating_sign[R](list_index_or_zero[I](tail, y).suc)) *
                                                            entry(head, x) * alternating_sign[R](list_index_or_zero[I](cs, x)) *
                                                            determinant_list[R, I](entry, tail.remove_one(y), cs.remove_one(x))
                                                    }))
                                                }))
                                        // Fubini: RHS-rest becomes sum over cs of a sum over tail
                                        sum_map_sum_exchange[I, R](tail, cs, function(y: I, x: I) {
                                            (entry(y, c) * alternating_sign[R](list_index_or_zero[I](tail, y).suc)) *
                                                entry(head, x) * alternating_sign[R](list_index_or_zero[I](cs, x)) *
                                                determinant_list[R, I](entry, tail.remove_one(y), cs.remove_one(x))
                                        })
                                        sum[R](map[I, R](cs, function(x: I) {
                                            sum[R](map[I, R](tail, function(y: I) {
                                                (entry(y, c) * alternating_sign[R](list_index_or_zero[I](tail, y).suc)) *
                                                    entry(head, x) * alternating_sign[R](list_index_or_zero[I](cs, x)) *
                                                    determinant_list[R, I](entry, tail.remove_one(y), cs.remove_one(x))
                                            }))
                                        })) = sum[R](map[I, R](tail, function(y: I) {
                                            sum[R](map[I, R](cs, function(x: I) {
                                                (entry(y, c) * alternating_sign[R](list_index_or_zero[I](tail, y).suc)) *
                                                    entry(head, x) * alternating_sign[R](list_index_or_zero[I](cs, x)) *
                                                    determinant_list[R, I](entry, tail.remove_one(y), cs.remove_one(x))
                                            }))
                                        }))
                                        determinant_list[R, I](flip_entry[R, I](entry), List.cons(c, cs), List.cons(head, tail)) =
                                            det_row_sum_term[R, I](flip_entry[R, I](entry), c, cs, List.cons(head, tail), head) +
                                                sum[R](map[I, R](cs, function(x: I) {
                                                    sum[R](map[I, R](tail, function(y: I) {
                                                        (entry(y, c) * alternating_sign[R](list_index_or_zero[I](tail, y).suc)) *
                                                            entry(head, x) * alternating_sign[R](list_index_or_zero[I](cs, x)) *
                                                            determinant_list[R, I](entry, tail.remove_one(y), cs.remove_one(x))
                                                    }))
                                                }))
                                        // pointwise: A(x)·B(y)·D = B'(y)·A'(x)·D
                                        forall(x: I) {
                                            if cs.contains(x) {
                                                forall(y: I) {
                                                    if tail.contains(y) {
                                                        alternating_sign_suc_comm[R](list_index_or_zero[I](cs, x), list_index_or_zero[I](tail, y))
                                                        alternating_sign[R](list_index_or_zero[I](cs, x).suc) * alternating_sign[R](list_index_or_zero[I](tail, y)) =
                                                            alternating_sign[R](list_index_or_zero[I](tail, y).suc) * alternating_sign[R](list_index_or_zero[I](cs, x))
                                                        entry(head, x) * entry(y, c) = entry(y, c) * entry(head, x)
                                                        prod_exchange_comm[R](entry(head, x), entry(y, c),
                                                            alternating_sign[R](list_index_or_zero[I](cs, x).suc), alternating_sign[R](list_index_or_zero[I](tail, y)),
                                                            alternating_sign[R](list_index_or_zero[I](tail, y).suc), alternating_sign[R](list_index_or_zero[I](cs, x)))
                                                        (entry(head, x) * alternating_sign[R](list_index_or_zero[I](cs, x).suc)) *
                                                            (entry(y, c) * alternating_sign[R](list_index_or_zero[I](tail, y))) =
                                                            (entry(y, c) * alternating_sign[R](list_index_or_zero[I](tail, y).suc)) *
                                                                (entry(head, x) * alternating_sign[R](list_index_or_zero[I](cs, x)))
                                                        (entry(head, x) * alternating_sign[R](list_index_or_zero[I](cs, x).suc)) *
                                                            (entry(y, c) * alternating_sign[R](list_index_or_zero[I](tail, y))) *
                                                            determinant_list[R, I](entry, tail.remove_one(y), cs.remove_one(x)) =
                                                            (entry(y, c) * alternating_sign[R](list_index_or_zero[I](tail, y).suc)) *
                                                                (entry(head, x) * alternating_sign[R](list_index_or_zero[I](cs, x))) *
                                                                determinant_list[R, I](entry, tail.remove_one(y), cs.remove_one(x))
                                                        (entry(head, x) * alternating_sign[R](list_index_or_zero[I](cs, x).suc)) *
                                                            entry(y, c) * alternating_sign[R](list_index_or_zero[I](tail, y)) *
                                                            determinant_list[R, I](entry, tail.remove_one(y), cs.remove_one(x)) =
                                                            (entry(y, c) * alternating_sign[R](list_index_or_zero[I](tail, y).suc)) *
                                                                entry(head, x) * alternating_sign[R](list_index_or_zero[I](cs, x)) *
                                                                determinant_list[R, I](entry, tail.remove_one(y), cs.remove_one(x))
                                                    }
                                                }
                                            }
                                        }
                                        forall(x: I) {
                                            if cs.contains(x) {
                                                sum_map_of_pointwise[I, R](tail, function(y: I) {
                                                    (entry(head, x) * alternating_sign[R](list_index_or_zero[I](cs, x).suc)) *
                                                        entry(y, c) * alternating_sign[R](list_index_or_zero[I](tail, y)) *
                                                        determinant_list[R, I](entry, tail.remove_one(y), cs.remove_one(x))
                                                }, function(y: I) {
                                                    (entry(y, c) * alternating_sign[R](list_index_or_zero[I](tail, y).suc)) *
                                                        entry(head, x) * alternating_sign[R](list_index_or_zero[I](cs, x)) *
                                                        determinant_list[R, I](entry, tail.remove_one(y), cs.remove_one(x))
                                                })
                                                sum[R](map[I, R](tail, function(y: I) {
                                                    (entry(head, x) * alternating_sign[R](list_index_or_zero[I](cs, x).suc)) *
                                                        entry(y, c) * alternating_sign[R](list_index_or_zero[I](tail, y)) *
                                                        determinant_list[R, I](entry, tail.remove_one(y), cs.remove_one(x))
                                                })) = sum[R](map[I, R](tail, function(y: I) {
                                                    (entry(y, c) * alternating_sign[R](list_index_or_zero[I](tail, y).suc)) *
                                                        entry(head, x) * alternating_sign[R](list_index_or_zero[I](cs, x)) *
                                                        determinant_list[R, I](entry, tail.remove_one(y), cs.remove_one(x))
                                                }))
                                            }
                                        }
                                        sum_map_of_pointwise[I, R](cs, function(x: I) {
                                            sum[R](map[I, R](tail, function(y: I) {
                                                (entry(head, x) * alternating_sign[R](list_index_or_zero[I](cs, x).suc)) *
                                                    entry(y, c) * alternating_sign[R](list_index_or_zero[I](tail, y)) *
                                                    determinant_list[R, I](entry, tail.remove_one(y), cs.remove_one(x))
                                            }))
                                        }, function(x: I) {
                                            sum[R](map[I, R](tail, function(y: I) {
                                                (entry(y, c) * alternating_sign[R](list_index_or_zero[I](tail, y).suc)) *
                                                    entry(head, x) * alternating_sign[R](list_index_or_zero[I](cs, x)) *
                                                    determinant_list[R, I](entry, tail.remove_one(y), cs.remove_one(x))
                                            }))
                                        })
                                        // LHS-rest equals RHS-rest (after Fubini)
                                        sum[R](map[I, R](cs, function(x: I) {
                                            sum[R](map[I, R](tail, function(y: I) {
                                                (entry(head, x) * alternating_sign[R](list_index_or_zero[I](cs, x).suc)) *
                                                    entry(y, c) * alternating_sign[R](list_index_or_zero[I](tail, y)) *
                                                    determinant_list[R, I](entry, tail.remove_one(y), cs.remove_one(x))
                                            }))
                                        })) = sum[R](map[I, R](cs, function(x: I) {
                                            sum[R](map[I, R](tail, function(y: I) {
                                                (entry(y, c) * alternating_sign[R](list_index_or_zero[I](tail, y).suc)) *
                                                    entry(head, x) * alternating_sign[R](list_index_or_zero[I](cs, x)) *
                                                    determinant_list[R, I](entry, tail.remove_one(y), cs.remove_one(x))
                                            }))
                                        }))
                                        // LHS reassembly: LHS = f(c) + LHS-rest
                                        determinant_list[R, I](entry, List.cons(head, tail), List.cons(c, cs)) =
                                            entry(head, c) * determinant_list[R, I](flip_entry[R, I](entry), cs, tail) +
                                                sum[R](map[I, R](cs, function(x: I) {
                                                    sum[R](map[I, R](tail, function(y: I) {
                                                        (entry(head, x) * alternating_sign[R](list_index_or_zero[I](cs, x).suc)) *
                                                            entry(y, c) * alternating_sign[R](list_index_or_zero[I](tail, y)) *
                                                            determinant_list[R, I](entry, tail.remove_one(y), cs.remove_one(x))
                                                    }))
                                                }))
                                        // RHS reassembly: RHS = g(head) + RHS-rest
                                        determinant_list[R, I](flip_entry[R, I](entry), List.cons(c, cs), List.cons(head, tail)) =
                                            entry(head, c) * determinant_list[R, I](flip_entry[R, I](entry), cs, tail) +
                                                sum[R](map[I, R](cs, function(x: I) {
                                                    sum[R](map[I, R](tail, function(y: I) {
                                                        (entry(y, c) * alternating_sign[R](list_index_or_zero[I](tail, y).suc)) *
                                                            entry(head, x) * alternating_sign[R](list_index_or_zero[I](cs, x)) *
                                                            determinant_list[R, I](entry, tail.remove_one(y), cs.remove_one(x))
                                                    }))
                                                }))
                                        determinant_list[R, I](entry, List.cons(head, tail), List.cons(c, cs)) =
                                            determinant_list[R, I](flip_entry[R, I](entry), List.cons(c, cs), List.cons(head, tail))

                                    }
                                }
                            }
                        }
                    }
                }
            }
        }
    }
}

/// The determinant-list expansion is invariant under exchanging the roles of
/// rows and columns: `det(entry, rows, cols) = det(flip_entry, cols, rows)`
/// for row and column lists of equal length.
/// The strong-induction step in the clean (non-expanded) form.
/// TODO(det_general): The strong-induction step `det_transpose_step` is fully
/// proved.  To finish the matrix-level theorem `det(A^T) = det(A)` one still needs
/// the prover to close the strong-induction step `true_below(pred, k) implies
/// pred(entry, k)` from the step theorem's conclusion.  The missing link is that
/// the search cannot re-derive a `forall`-shaped goal whose conclusion is the
/// determinant equality from the closed `forall`-block (the case analysis is not
/// replayed); equivalently, one needs a prover improvement (or a manual proof of
/// the forall-instantiation) connecting `det_transpose_step`'s conclusion to the
/// predicate `det_transpose_pred`.  Once that connects, `strong_induction` over
/// `det_transpose_pred` and the per-instance instantiation below complete the proof:
///
///     forall(k: Nat) { det_transpose_step(entry, k) }
///     strong_induction(det_transpose_pred(entry))
///     det_transpose_pred(entry, rows.length)
///     ... unfold pred at rows.length, instantiate at (rows, cols) ...
///


/// Successor equality cancels.

/// Exchanging the middle factors of a four-fold product.

/// The product of the position-wise diagonal pairs of two row and column lists.
define determinant_diagonal[R: Ring, I](entry: (I, I) -> R, rows: List[I], cols: List[I]) -> R {
    match rows {
        List.nil {
            R.1
        }
        List.cons(row, tail_rows) {
            match cols {
                List.nil {
                    R.1
                }
                List.cons(col, tail_cols) {
                    entry(row, col) * determinant_diagonal[R, I](entry, tail_rows, tail_cols)
                }
            }
        }
    }
}

/// An element with index zero is the head of the list.
theorem list_index_or_zero_zero_iff_head[I](head: I, tail: List[I], x: I) {
    List.cons(head, tail).contains(x) and list_index_or_zero[I](List.cons(head, tail), x) = Nat.0 implies x = head
} by {
    if List.cons(head, tail).contains(x) and list_index_or_zero[I](List.cons(head, tail), x) = Nat.0 {
        if x = head {
            x = head
        } else {
            list_index_or_zero_cons_other[I](head, tail, x)
            list_index_or_zero[I](List.cons(head, tail), x) = list_index_or_zero[I](tail, x).suc
            Nat.0 = list_index_or_zero[I](tail, x).suc
            false
        }
    }
}

/// Removing an element before another leaves the other's index unchanged.
theorem list_index_or_zero_remove_after[I](list: List[I], x: I, c: I) {
    list.contains(x) and list.contains(c) and x != c and
        list_index_or_zero[I](list, c) < list_index_or_zero[I](list, x) implies
        list_index_or_zero[I](list.remove_one(x), c) = list_index_or_zero[I](list, c)
} by {
    define p(l: List[I]) -> Bool {
        l.contains(x) and l.contains(c) and x != c and
            list_index_or_zero[I](l, c) < list_index_or_zero[I](l, x) implies
            list_index_or_zero[I](l.remove_one(x), c) = list_index_or_zero[I](l, c)
    }
    p(List.nil[I])
    forall(head: I, tail: List[I]) {
        if p(tail) {
            if List.cons(head, tail).contains(x) and List.cons(head, tail).contains(c) and x != c and
                list_index_or_zero[I](List.cons(head, tail), c) < list_index_or_zero[I](List.cons(head, tail), x) {
                if head = x {
                    // removing the head: c is after x
                    remove_one_cons_eq(head, tail)
                    List.cons(head, tail).remove_one(x) = tail
                    // idx(cons(head,tail), c) = 1 + idx(tail, c) since c != head = x
                    head != c
                    list_index_or_zero_cons_other[I](head, tail, c)
                    list_index_or_zero[I](List.cons(head, tail), c) = list_index_or_zero[I](tail, c).suc
                    list_index_or_zero[I](tail, c) < list_index_or_zero[I](List.cons(head, tail), x)
                    list_index_or_zero_cons_self[I](head, tail)
                    list_index_or_zero[I](List.cons(head, tail), x) = Nat.0
                    list_index_or_zero[I](tail, c).suc < Nat.0
                    false
                }
                if head != x {
                    remove_one_cons_neq[I](head, tail, x)
                    List.cons(head, tail).remove_one(x) = List.cons(head, tail.remove_one(x))
                    list_index_or_zero[I](List.cons(head, tail).remove_one(x), c) =
                        list_index_or_zero[I](List.cons(head, tail.remove_one(x)), c)
                    if head = c {
                        list_index_or_zero_cons_self[I](head, tail.remove_one(x))
                        list_index_or_zero[I](List.cons(head, tail.remove_one(x)), c) = Nat.0
                        list_index_or_zero_cons_self[I](head, tail)
                        list_index_or_zero[I](List.cons(head, tail), c) = Nat.0
                        list_index_or_zero[I](List.cons(head, tail).remove_one(x), c) = Nat.0
                        list_index_or_zero[I](List.cons(head, tail).remove_one(x), c) =
                            list_index_or_zero[I](List.cons(head, tail), c)
                    }
                    if head != c {
                        list_index_or_zero_cons_other[I](head, tail, c)
                        list_index_or_zero[I](List.cons(head, tail), c) = list_index_or_zero[I](tail, c).suc
                        list_index_or_zero_cons_other[I](head, tail, x)
                        list_index_or_zero[I](List.cons(head, tail), x) = list_index_or_zero[I](tail, x).suc
                        list_index_or_zero[I](tail, c).suc < list_index_or_zero[I](tail, x).suc
                        lt_cancel_suc(list_index_or_zero[I](tail, c), list_index_or_zero[I](tail, x))
                        list_index_or_zero[I](tail, c) < list_index_or_zero[I](tail, x)
                        tail.contains(x)
                        tail.contains(c)
                        x != c
                        p(tail)
                        list_index_or_zero[I](tail.remove_one(x), c) = list_index_or_zero[I](tail, c)
                        list_index_or_zero_cons_other[I](head, tail.remove_one(x), c)
                        head != c
                        list_index_or_zero[I](List.cons(head, tail.remove_one(x)), c) =
                            list_index_or_zero[I](tail.remove_one(x), c).suc
                        list_index_or_zero[I](List.cons(head, tail.remove_one(x)), c) =
                            list_index_or_zero[I](tail, c).suc
                        list_index_or_zero[I](List.cons(head, tail).remove_one(x), c) =
                            list_index_or_zero[I](List.cons(head, tail), c)
                    }
                    list_index_or_zero[I](List.cons(head, tail).remove_one(x), c) =
                        list_index_or_zero[I](List.cons(head, tail), c)
                }
                list_index_or_zero[I](List.cons(head, tail).remove_one(x), c) =
                    list_index_or_zero[I](List.cons(head, tail), c)
            }
            p(List.cons(head, tail))
        }
    }
    p(List.nil[I]) and forall(head: I, tail: List[I]) { p(tail) implies p(List.cons(head, tail)) }
    List.induction(p)
    forall(l: List[I]) { p(l) }
    p(list)
    if list.contains(x) and list.contains(c) and x != c and
        list_index_or_zero[I](list, c) < list_index_or_zero[I](list, x) {
        list_index_or_zero[I](list.remove_one(x), c) = list_index_or_zero[I](list, c)
    }
}

/// Removing an element after another shifts the other's index down by one.
theorem list_index_or_zero_remove_before[I](list: List[I], x: I, c: I) {
    list.contains(x) and list.contains(c) and x != c and
        list_index_or_zero[I](list, x) < list_index_or_zero[I](list, c) implies
        list_index_or_zero[I](list.remove_one(x), c).suc = list_index_or_zero[I](list, c)
} by {
    define p(l: List[I]) -> Bool {
        l.contains(x) and l.contains(c) and x != c and
            list_index_or_zero[I](l, x) < list_index_or_zero[I](l, c) implies
            list_index_or_zero[I](l.remove_one(x), c).suc = list_index_or_zero[I](l, c)
    }
    p(List.nil[I])
    forall(head: I, tail: List[I]) {
        if p(tail) {
            if List.cons(head, tail).contains(x) and List.cons(head, tail).contains(c) and x != c and
                list_index_or_zero[I](List.cons(head, tail), x) < list_index_or_zero[I](List.cons(head, tail), c) {
                if head = x {
                    remove_one_cons_eq(head, tail)
                    List.cons(head, tail).remove_one(x) = tail
                    head != c
                    list_index_or_zero_cons_other[I](head, tail, c)
                    list_index_or_zero[I](List.cons(head, tail), c) = list_index_or_zero[I](tail, c).suc
                    list_index_or_zero_cons_self[I](head, tail)
                    list_index_or_zero[I](List.cons(head, tail), x) = Nat.0
                    Nat.0 < list_index_or_zero[I](tail, c).suc
                    list_index_or_zero[I](List.cons(head, tail).remove_one(x), c).suc =
                        list_index_or_zero[I](tail, c).suc
                    list_index_or_zero[I](List.cons(head, tail).remove_one(x), c).suc =
                        list_index_or_zero[I](List.cons(head, tail), c)
                }
                if head != x {
                    remove_one_cons_neq[I](head, tail, x)
                    List.cons(head, tail).remove_one(x) = List.cons(head, tail.remove_one(x))
                    if head = c {
                        // x after head = c: idx(rem, c) = 0 = idx(c)... but x before c means idx(x) < 0 — impossible
                        list_index_or_zero_cons_self[I](head, tail)
                        list_index_or_zero[I](List.cons(head, tail), c) = Nat.0
                        list_index_or_zero[I](List.cons(head, tail), x) < Nat.0
                        false
                    }
                    if head != c {
                        list_index_or_zero_cons_other[I](head, tail, c)
                        list_index_or_zero[I](List.cons(head, tail), c) = list_index_or_zero[I](tail, c).suc
                        list_index_or_zero_cons_other[I](head, tail, x)
                        list_index_or_zero[I](List.cons(head, tail), x) = list_index_or_zero[I](tail, x).suc
                        list_index_or_zero[I](tail, x).suc < list_index_or_zero[I](tail, c).suc
                        lt_cancel_suc(list_index_or_zero[I](tail, x), list_index_or_zero[I](tail, c))
                        list_index_or_zero[I](tail, x) < list_index_or_zero[I](tail, c)
                        tail.contains(x)
                        tail.contains(c)
                        x != c
                        p(tail)
                        list_index_or_zero[I](tail.remove_one(x), c).suc = list_index_or_zero[I](tail, c)
                        list_index_or_zero_cons_other[I](head, tail.remove_one(x), c)
                        head != c
                        list_index_or_zero[I](List.cons(head, tail.remove_one(x)), c) =
                            list_index_or_zero[I](tail.remove_one(x), c).suc
                        list_index_or_zero[I](List.cons(head, tail.remove_one(x)), c).suc =
                            list_index_or_zero[I](tail.remove_one(x), c).suc.suc
                        list_index_or_zero[I](List.cons(head, tail.remove_one(x)), c).suc =
                            list_index_or_zero[I](tail, c).suc
                        list_index_or_zero[I](List.cons(head, tail).remove_one(x), c).suc =
                            list_index_or_zero[I](List.cons(head, tail), c)
                    }
                    list_index_or_zero[I](List.cons(head, tail).remove_one(x), c).suc =
                        list_index_or_zero[I](List.cons(head, tail), c)
                }
                list_index_or_zero[I](List.cons(head, tail).remove_one(x), c).suc =
                    list_index_or_zero[I](List.cons(head, tail), c)
            }
            p(List.cons(head, tail))
        }
    }
    p(List.nil[I]) and forall(head: I, tail: List[I]) { p(tail) implies p(List.cons(head, tail)) }
    List.induction(p)
    forall(l: List[I]) { p(l) }
    p(list)
    if list.contains(x) and list.contains(c) and x != c and
        list_index_or_zero[I](list, x) < list_index_or_zero[I](list, c) {
        list_index_or_zero[I](list.remove_one(x), c).suc = list_index_or_zero[I](list, c)
    }
}

/// In a unique list, equal indices determine equal elements.
theorem list_index_or_zero_injective_unique[I](list: List[I], x: I, y: I) {
    list.is_unique and list.contains(x) and list.contains(y) and
        list_index_or_zero[I](list, x) = list_index_or_zero[I](list, y) implies x = y
} by {
    define p(l: List[I]) -> Bool {
        l.is_unique and l.contains(x) and l.contains(y) and
            list_index_or_zero[I](l, x) = list_index_or_zero[I](l, y) implies x = y
    }
    p(List.nil[I])
    forall(head: I, tail: List[I]) {
        if p(tail) {
            if List.cons(head, tail).is_unique and List.cons(head, tail).contains(x) and List.cons(head, tail).contains(y) and
                list_index_or_zero[I](List.cons(head, tail), x) = list_index_or_zero[I](List.cons(head, tail), y) {
                if x = head {
                    list_index_or_zero_cons_self[I](head, tail)
                    list_index_or_zero[I](List.cons(head, tail), x) = Nat.0
                    list_index_or_zero[I](List.cons(head, tail), y) = Nat.0
                    list_index_or_zero_zero_iff_head[I](head, tail, y)
                    y = head
                    x = y
                }
                if x != head {
                    list_index_or_zero_cons_other[I](head, tail, x)
                    list_index_or_zero[I](List.cons(head, tail), x) = list_index_or_zero[I](tail, x).suc
                    list_index_or_zero[I](List.cons(head, tail), x) = list_index_or_zero[I](List.cons(head, tail), y)
                    list_index_or_zero[I](List.cons(head, tail), y) = list_index_or_zero[I](tail, x).suc
                    if y = head {
                        list_index_or_zero_cons_self[I](head, tail)
                        list_index_or_zero[I](List.cons(head, tail), y) = Nat.0
                        list_index_or_zero[I](tail, x).suc = Nat.0
                        false
                    }
                    if y != head {
                        list_index_or_zero_cons_other[I](head, tail, y)
                        list_index_or_zero[I](List.cons(head, tail), y) = list_index_or_zero[I](tail, y).suc
                        list_index_or_zero[I](tail, x).suc = list_index_or_zero[I](tail, y).suc
                        list_index_or_zero[I](tail, x) = list_index_or_zero[I](tail, y)
                        unique_implies_tail_unique[I](head, tail)
                        List.cons(head, tail).is_unique implies tail.is_unique
                        tail.is_unique
                        tail.contains(x)
                        tail.contains(y)
                        p(tail)
                        x = y
                    }
                    x = y
                }
                x = y
            }
            p(List.cons(head, tail))
        }
    }
    p(List.nil[I]) and forall(head: I, tail: List[I]) { p(tail) implies p(List.cons(head, tail)) }
    List.induction(p)
    forall(l: List[I]) { p(l) }
    p(list)
    if list.is_unique and list.contains(x) and list.contains(y) and
        list_index_or_zero[I](list, x) = list_index_or_zero[I](list, y) {
        x = y
    }
}


/// The upper-triangular condition: entries below the diagonal (with respect to
/// the given row and column orderings) vanish.
define triangular_condition[R: Ring, I](entry: (I, I) -> R, rows: List[I], cols: List[I]) -> Bool {
    forall(r: I, c: I) {
        rows.contains(r) and cols.contains(c) and
        list_index_or_zero[I](rows, r) > list_index_or_zero[I](cols, c) implies entry(r, c) = R.0
    }
}

/// Removing the first row and column preserves the upper-triangular condition.
/// Removing the first row and column preserves the upper-triangular condition.
theorem triangular_preserved_first[R: Ring, I](entry: (I, I) -> R, head: I, tail: List[I], c: I, cs: List[I]) {
    List.cons(head, tail).is_unique and List.cons(c, cs).is_unique and
    triangular_condition[R, I](entry, List.cons(head, tail), List.cons(c, cs)) implies
        triangular_condition[R, I](entry, tail, cs)
} by {
    if List.cons(head, tail).is_unique and List.cons(c, cs).is_unique and
        triangular_condition[R, I](entry, List.cons(head, tail), List.cons(c, cs)) {
        triangular_condition[R, I](entry, List.cons(head, tail), List.cons(c, cs)) = (forall(r: I, c2: I) {
            List.cons(head, tail).contains(r) and List.cons(c, cs).contains(c2) and
            list_index_or_zero[I](List.cons(head, tail), r) > list_index_or_zero[I](List.cons(c, cs), c2) implies entry(r, c2) = R.0
        })
        forall(r: I, c2: I) {
            List.cons(head, tail).contains(r) and List.cons(c, cs).contains(c2) and
            list_index_or_zero[I](List.cons(head, tail), r) > list_index_or_zero[I](List.cons(c, cs), c2) implies entry(r, c2) = R.0
        }
        forall(r: I, c2: I) {
            if tail.contains(r) and cs.contains(c2) and
                list_index_or_zero[I](tail, r) > list_index_or_zero[I](cs, c2) {
                if r = head {
                    unique_cons_not_contains_head[I](head, tail)
                    not tail.contains(head)
                    tail.contains(r) = tail.contains(head)
                    tail.contains(r) = false
                    false
                }
                r != head
                list_index_or_zero_cons_other[I](head, tail, r)
                list_index_or_zero[I](List.cons(head, tail), r) = list_index_or_zero[I](tail, r).suc
                if c2 = c {
                    unique_cons_not_contains_head[I](c, cs)
                    not cs.contains(c)
                    cs.contains(c2) = cs.contains(c)
                    cs.contains(c2) = false
                    false
                }
                c2 != c
                list_index_or_zero_cons_other[I](c, cs, c2)
                list_index_or_zero[I](List.cons(c, cs), c2) = list_index_or_zero[I](cs, c2).suc
                lt_add_left(Nat.1, list_index_or_zero[I](cs, c2), list_index_or_zero[I](tail, r))
                Nat.1 + list_index_or_zero[I](cs, c2) < Nat.1 + list_index_or_zero[I](tail, r)
                list_index_or_zero[I](List.cons(c, cs), c2) < list_index_or_zero[I](List.cons(head, tail), r)
                list_index_or_zero[I](List.cons(head, tail), r) > list_index_or_zero[I](List.cons(c, cs), c2)
                List.cons(head, tail).contains(r)
                List.cons(c, cs).contains(c2)
                entry(r, c2) = R.0
            }
        }
        triangular_condition[R, I](entry, tail, cs) = (forall(r: I, c2: I) {
            tail.contains(r) and cs.contains(c2) and
            list_index_or_zero[I](tail, r) > list_index_or_zero[I](cs, c2) implies entry(r, c2) = R.0
        })
        triangular_condition[R, I](entry, tail, cs)
    }
}
/// Removing a column other than the first preserves the upper-triangular condition.
theorem triangular_preserved_remove[R: Ring, I](entry: (I, I) -> R, head: I, tail: List[I], cols: List[I], x: I) {
    List.cons(head, tail).is_unique and cols.is_unique and
    triangular_condition[R, I](entry, List.cons(head, tail), cols) and cols.contains(x) implies
        triangular_condition[R, I](entry, tail, cols.remove_one(x))
} by {
    if List.cons(head, tail).is_unique and cols.is_unique and
        triangular_condition[R, I](entry, List.cons(head, tail), cols) and cols.contains(x) {
        triangular_condition[R, I](entry, List.cons(head, tail), cols) = (forall(r: I, c: I) {
            List.cons(head, tail).contains(r) and cols.contains(c) and
            list_index_or_zero[I](List.cons(head, tail), r) > list_index_or_zero[I](cols, c) implies entry(r, c) = R.0
        })
        forall(r: I, c: I) {
            List.cons(head, tail).contains(r) and cols.contains(c) and
            list_index_or_zero[I](List.cons(head, tail), r) > list_index_or_zero[I](cols, c) implies entry(r, c) = R.0
        }
        forall(r: I, c: I) {
            if tail.contains(r) and cols.remove_one(x).contains(c) and
                list_index_or_zero[I](tail, r) > list_index_or_zero[I](cols.remove_one(x), c) {
                if r = head {
                    unique_cons_not_contains_head[I](head, tail)
                    not tail.contains(head)
                    tail.contains(r) = tail.contains(head)
                    tail.contains(r) = false
                    false
                }
                r != head
                list_index_or_zero_cons_other[I](head, tail, r)
                list_index_or_zero[I](List.cons(head, tail), r) = list_index_or_zero[I](tail, r).suc
                remove_one_unique_not_contains_self[I](cols, x)
                not cols.remove_one(x).contains(x)
                if c = x {
                    cols.remove_one(x).contains(c) = cols.remove_one(x).contains(x)
                    cols.remove_one(x).contains(c) = false
                    false
                }
                c != x
                list_index_or_zero_injective_unique[I](cols, x, c)
                list_index_or_zero[I](cols, x) != list_index_or_zero[I](cols, c)
                if list_index_or_zero[I](cols, c) < list_index_or_zero[I](cols, x) {
                    list_index_or_zero_remove_after[I](cols, x, c)
                    list_index_or_zero[I](cols.remove_one(x), c) = list_index_or_zero[I](cols, c)
                    list_index_or_zero[I](tail, r) > list_index_or_zero[I](cols, c)
                    lt_add_left(Nat.1, list_index_or_zero[I](cols, c), list_index_or_zero[I](tail, r))
                    Nat.1 + list_index_or_zero[I](cols, c) < Nat.1 + list_index_or_zero[I](tail, r)
                    lt_suc(list_index_or_zero[I](cols, c))
                    list_index_or_zero[I](cols, c) < list_index_or_zero[I](cols, c).suc
                    list_index_or_zero[I](cols, c).suc = Nat.1 + list_index_or_zero[I](cols, c)
                    list_index_or_zero[I](cols, c) < Nat.1 + list_index_or_zero[I](cols, c)
                    lt_trans(list_index_or_zero[I](cols, c), Nat.1 + list_index_or_zero[I](cols, c), Nat.1 + list_index_or_zero[I](tail, r))
                    list_index_or_zero[I](cols, c) < Nat.1 + list_index_or_zero[I](tail, r)
                    list_index_or_zero[I](cols, c) < list_index_or_zero[I](List.cons(head, tail), r)
                    list_index_or_zero[I](List.cons(head, tail), r) > list_index_or_zero[I](cols, c)
                    List.cons(head, tail).contains(r)
                    cols.contains(c)
                    entry(r, c) = R.0
                }
                if not list_index_or_zero[I](cols, c) < list_index_or_zero[I](cols, x) {
                    if list_index_or_zero[I](cols, x) < list_index_or_zero[I](cols, c) {
                        list_index_or_zero_remove_before[I](cols, x, c)
                        list_index_or_zero[I](cols.remove_one(x), c).suc = list_index_or_zero[I](cols, c)
                        list_index_or_zero[I](tail, r) > list_index_or_zero[I](cols.remove_one(x), c)
                        lt_add_left(Nat.1, list_index_or_zero[I](cols.remove_one(x), c), list_index_or_zero[I](tail, r))
                        Nat.1 + list_index_or_zero[I](cols.remove_one(x), c) < Nat.1 + list_index_or_zero[I](tail, r)
                        list_index_or_zero[I](cols.remove_one(x), c).suc < Nat.1 + list_index_or_zero[I](tail, r)
                        list_index_or_zero[I](cols, c) < Nat.1 + list_index_or_zero[I](tail, r)
                        list_index_or_zero[I](cols, c) < list_index_or_zero[I](List.cons(head, tail), r)
                        list_index_or_zero[I](List.cons(head, tail), r) > list_index_or_zero[I](cols, c)
                        List.cons(head, tail).contains(r)
                        cols.contains(c)
                        entry(r, c) = R.0
                    }
                    if not list_index_or_zero[I](cols, x) < list_index_or_zero[I](cols, c) {
                        list_index_or_zero[I](cols, x) = list_index_or_zero[I](cols, c)
                        false
                    }
                    entry(r, c) = R.0
                }
                entry(r, c) = R.0
            }
        }
        triangular_condition[R, I](entry, tail, cols.remove_one(x)) = (forall(r: I, c: I) {
            tail.contains(r) and cols.remove_one(x).contains(c) and
            list_index_or_zero[I](tail, r) > list_index_or_zero[I](cols.remove_one(x), c) implies entry(r, c) = R.0
        })
        triangular_condition[R, I](entry, tail, cols.remove_one(x))
    }
}

/// The determinant-list expansion of an upper-triangular matrix is the product
/// of its diagonal entries.
/// The determinant-list expansion of an upper-triangular matrix is the product
/// of its diagonal entries.
/// TODO(det_general): The upper-triangular determinant theorem is fully proved
/// up to the final `List.induction` step.  The step lemma's case analysis (the
/// `match cols2` block inside the `forall(cols2)` induction step) verifies, as do
/// the helpers `triangular_condition`, `determinant_diagonal`, the index lemmas
/// and the preservation lemmas `triangular_preserved_first`/`triangular_preserved_remove`.
/// What remains is closing the induction-step goal `p(List.cons(head, tail))` (a
/// `forall`-shaped goal whose conclusion is the determinant/diagonal equality); the
/// current prover search cannot re-derive such goals from the closed `forall`-block.
/// Once that connects, the statement to instantiate is:
///
///     forall(rows, cols) { rows.length = cols.length and rows.is_unique and
///         cols.is_unique and triangular_condition(entry, rows, cols) implies
///         determinant_list(entry, rows, cols) = determinant_diagonal(entry, rows, cols) }
///
/// and the matrix corollary `matrix_det_suc_upper_triangular` follows by unfolding
/// `matrix_det_suc` and using `fin_enum_suc_get_idx` (positions equal values).

/// Removing two elements from a list commutes.
theorem remove_one_comm[T](list: List[T], x: T, y: T) {
    list.remove_one(x).remove_one(y) = list.remove_one(y).remove_one(x)
} by {
    define p(l: List[T]) -> Bool {
        l.remove_one(x).remove_one(y) = l.remove_one(y).remove_one(x)
    }
    p(List.nil[T])
    forall(head: T, tail: List[T]) {
        if p(tail) {
            if head = x {
                if head = y {
                    remove_one_cons_eq(head, tail)
                    List.cons(head, tail).remove_one(x) = tail
                    List.cons(head, tail).remove_one(y) = tail
                    x = y
                    List.cons(head, tail).remove_one(x).remove_one(y) =
                        List.cons(head, tail).remove_one(y).remove_one(x)
                }
                if head != y {
                    remove_one_cons_eq(head, tail)
                    List.cons(head, tail).remove_one(x) = tail
                    remove_one_cons_neq[T](head, tail, y)
                    List.cons(head, tail).remove_one(y) = List.cons(head, tail.remove_one(y))
                    List.cons(head, tail).remove_one(x).remove_one(y) = tail.remove_one(y)
                    List.cons(head, tail).remove_one(y).remove_one(x) =
                        List.cons(head, tail.remove_one(y)).remove_one(x)
                    remove_one_cons_eq(head, tail.remove_one(y))
                    List.cons(head, tail.remove_one(y)).remove_one(x) = tail.remove_one(y)
                    List.cons(head, tail).remove_one(x).remove_one(y) =
                        List.cons(head, tail).remove_one(y).remove_one(x)
                }
                List.cons(head, tail).remove_one(x).remove_one(y) =
                    List.cons(head, tail).remove_one(y).remove_one(x)
            }
            if head != x {
                remove_one_cons_neq[T](head, tail, x)
                List.cons(head, tail).remove_one(x) = List.cons(head, tail.remove_one(x))
                if head = y {
                    remove_one_cons_eq(head, tail)
                    List.cons(head, tail).remove_one(y) = tail
                    List.cons(head, tail).remove_one(x).remove_one(y) =
                        List.cons(head, tail.remove_one(x)).remove_one(y)
                    remove_one_cons_eq(head, tail.remove_one(x))
                    List.cons(head, tail.remove_one(x)).remove_one(y) = tail.remove_one(x)
                    List.cons(head, tail).remove_one(y).remove_one(x) = tail.remove_one(x)
                    List.cons(head, tail).remove_one(x).remove_one(y) =
                        List.cons(head, tail).remove_one(y).remove_one(x)
                }
                if head != y {
                    remove_one_cons_neq[T](head, tail, y)
                    List.cons(head, tail).remove_one(y) = List.cons(head, tail.remove_one(y))
                    List.cons(head, tail).remove_one(x).remove_one(y) =
                        List.cons(head, tail.remove_one(x)).remove_one(y)
                    remove_one_cons_neq[T](head, tail.remove_one(x), y)
                    List.cons(head, tail.remove_one(x)).remove_one(y) =
                        List.cons(head, tail.remove_one(x).remove_one(y))
                    List.cons(head, tail).remove_one(y).remove_one(x) =
                        List.cons(head, tail.remove_one(y)).remove_one(x)
                    remove_one_cons_neq[T](head, tail.remove_one(y), x)
                    List.cons(head, tail.remove_one(y)).remove_one(x) =
                        List.cons(head, tail.remove_one(y).remove_one(x))
                    p(tail)
                    tail.remove_one(x).remove_one(y) = tail.remove_one(y).remove_one(x)
                    List.cons(head, tail).remove_one(x).remove_one(y) =
                        List.cons(head, tail).remove_one(y).remove_one(x)
                }
                List.cons(head, tail).remove_one(x).remove_one(y) =
                    List.cons(head, tail).remove_one(y).remove_one(x)
            }
            List.cons(head, tail).remove_one(x).remove_one(y) =
                List.cons(head, tail).remove_one(y).remove_one(x)
        }
    }
    p(List.nil[T]) and forall(head: T, tail: List[T]) { p(tail) implies p(List.cons(head, tail)) }
    List.induction(p)
    forall(l: List[T]) { p(l) }
    p(list)
    list.remove_one(x).remove_one(y) = list.remove_one(y).remove_one(x)
}

/// Row-swap negation (task 2) — DONE at the determinant-list level.
///
/// The general-n adjacent-swap identity is proved as
/// `determinant_list_swap_first_two_rows`:
///
///     determinant_list(entry, List.cons(r2, List.cons(r1, rs)), cols) =
///         -(determinant_list(entry, List.cons(r1, List.cons(r2, rs)), cols))
///
/// over a CommRing, with the hypothesis `cols.is_unique`: the double sums range
/// over the ordered pairs (x, y) with x != y, and the reindexing (x, y) -> (y, x)
/// is a bijection on that pair set only when columns are unique — with duplicates
/// the two double sums are not even indexed by the same pair set.  The proof expands both sides along their two head rows
/// (`determinant_list_two_rows`), giving double sums over ordered pairs
/// (x, y) with x != y:
///
///     LHS = sum_{x in cols} sum_{y in cols.remove_one(x)}
///             entry(r2,x)*alt(idx_cols(x))*entry(r1,y)*alt(idx_{rem x}(y)) * D(x,y)
///     RHS = sum_{x in cols} sum_{y in cols.remove_one(x)}
///             entry(r1,x)*alt(idx_cols(x))*entry(r2,y)*alt(idx_{rem x}(y)) * D(x,y)
///
/// where D(x,y) = determinant_list(entry, rs, cols.remove_one(x).remove_one(y)).
/// The machinery, in order:
///   * `sum_double_remove_flip` reindexes a double sum by (x, y) -> (y, x) over
///     a unique list (the ordered pairs of distinct elements);
///   * `det_swap_sign_pair` proves the sign bracket antisymmetry
///     alt(a)*alt(b') = -(alt(b)*alt(a')) with a = idx_cols(x), b = idx_cols(y),
///     b' = idx_{rem x}(y), a' = idx_{rem y}(x), by case-splitting on
///     a < b (trichotomy) and using list_index_or_zero_remove_before/after,
///     alternating_sign_suc and mul_neg_left (no alternating_sign_add needed);
///   * `swap_term_pair_neg` pairs the summands: T(r2, r1)(x, y) = -T(r1, r2)(y, x),
///     using det_swap_sign_pair, remove_one_comm (D(x,y) = D(y,x)) and CommRing
///     commutation of the entries;
///   * `sum_map_neg` pulls a negation out of a mapped sum.
///
/// TODO(det_general): The matrix theorem `matrix_det_suc_swap_first_two_rows`
/// (matrix_det_suc of matrix_swap_rows(0, 1) negates) needs the head structure of
/// fin_enum_suc(n) = map(n.suc.range, fin_of_nat_suc(n)): with enum = cons(0,
/// cons(1, rs)), the theorem applies with entry_S = the swapped matrix's entries,
/// and det(entry_S, [1, 0, rs], cols) = det(entry_A, [0, 1, rs], cols) by
/// comparing the two-row expansions pointwise via matrix_entry_swap_rows.  The
/// missing piece is peeling `cons` heads off map(range, fin_of_nat_suc) (range is
/// built by appending, so one needs range cons-structure lemmas).
///
/// TODO(det_general): Adjacent swaps deeper in the row list follow from this
/// theorem by peeling the rows before the pair: after k head-row expansions the
/// residual determinant is det(entry, cons(r_{k+1}, cons(r_k, rs')), cols') with
/// cols' the column list minus the first k columns, and the same proof applies
/// with (cols, rs) := (cols', rs').

/// TODO(det_general) — multiplicativity det(AB) = det(A)*det(B) (task 3).
///
/// Recommended strategy: prove that `det` is the unique alternating multilinear
/// function on rows with det(I) = 1, then compare the alternating multilinear
/// functions B -> det(A*B) and B -> det(A)*det(B), which agree at B = I.
/// Required lemmas, in order:
///   1. first-row multilinearity: with the head row expressed via
///      determinant_list_cons_rows_term, sums split and scalars factor out
///      (sum_map_add_fn, sum_map_scalar_mul are already proved);
///   2. row-swap negation (see above), which gives alternation and moves
///      multilinearity to any row;
///   3. the Leibniz expansion det = sum over row-index assignments of the signed
///      products, restricted to permutations by alternation — this is the heavy
///      piece and needs sums over all functions from rows to columns;
///   4. equal-rows-vanish (det with two equal adjacent rows is 0) via the
///      adjacent-swap cancellation pairing (signs cancel per column pair).
/// The 2x2 case (matrix_det_suc_two_by_two_mul) is already proved and serves as
/// the base check.

/// TODO(det_general) — invertibility criterion (task 5).
///
/// The adjugate route: define the cofactor matrix C with entries
/// matrix_cofactor_sign * det(minor), prove A * adj(A) = det(A) * I by expanding
/// the product entries and using Laplace expansion along a row (the cofactor
/// machinery matrix_minor / matrix_cofactor_sign / matrix_laplace_row already
/// exists in fin_matrix.ac), then conclude det(A) != 0 implies invertibility with
/// inverse adj(A)/det(A).  This depends on the full determinant theory above
/// (transpose, row-swap, multiplicativity) and on the Laplace-expansion-equals-
/// determinant lemma, so it is naturally last.

/// Products of alternating signs add their indices.
theorem alternating_sign_add[R: Ring](m: Nat, n: Nat) {
    alternating_sign[R](m) * alternating_sign[R](n) = alternating_sign[R](m + n)
} by {
    define p(k: Nat) -> Bool {
        alternating_sign[R](k) * alternating_sign[R](n) = alternating_sign[R](k + n)
    }
    alternating_sign_zero[R]
    alternating_sign[R](Nat.0) = R.1
    R.1 * alternating_sign[R](n) = alternating_sign[R](n)
    alternating_sign[R](Nat.0 + n) = alternating_sign[R](n)
    p(Nat.0)
    forall(k: Nat) {
        if p(k) {
            alternating_sign_suc[R](k)
            alternating_sign[R](k.suc) = -alternating_sign[R](k)
            alternating_sign[R](k.suc) * alternating_sign[R](n) = (-alternating_sign[R](k)) * alternating_sign[R](n)
            mul_neg_left[R](alternating_sign[R](k), alternating_sign[R](n))
            (-alternating_sign[R](k)) * alternating_sign[R](n) = -(alternating_sign[R](k) * alternating_sign[R](n))
            alternating_sign[R](k) * alternating_sign[R](n) = alternating_sign[R](k + n)
            -(alternating_sign[R](k) * alternating_sign[R](n)) = -(alternating_sign[R](k + n))
            add_suc_left(k, n)
            k.suc + n = (k + n).suc
            alternating_sign_suc[R](k + n)
            alternating_sign[R]((k + n).suc) = -alternating_sign[R](k + n)
            alternating_sign[R](k.suc + n) = -alternating_sign[R](k + n)
            alternating_sign[R](k.suc) * alternating_sign[R](n) = alternating_sign[R](k.suc + n)
            p(k.suc)
        }
    }
    p(Nat.0) and forall(k: Nat) { p(k) implies p(k.suc) }
    Nat.induction(p)
    p(m)
    alternating_sign[R](m) * alternating_sign[R](n) = alternating_sign[R](m + n)
}

/// Swapping the roles of the two indices in a double sum over pairs of distinct
/// elements of a unique list.
theorem sum_double_remove_flip[T, R: Ring](cols: List[T], f: T -> T -> R) {
    cols.is_unique implies
    (sum[R](map[T, R](cols, function(x: T) {
        sum[R](map[T, R](cols.remove_one(x), function(y: T) { f(x, y) }))
    })) = sum[R](map[T, R](cols, function(x: T) {
        sum[R](map[T, R](cols.remove_one(x), function(y: T) { f(y, x) }))
    })))
} by {
    define p(l: List[T]) -> Bool {
        l.is_unique implies
        (sum[R](map[T, R](l, function(x: T) {
            sum[R](map[T, R](l.remove_one(x), function(y: T) { f(x, y) }))
        })) = sum[R](map[T, R](l, function(x: T) {
            sum[R](map[T, R](l.remove_one(x), function(y: T) { f(y, x) }))
        })))
    }
    map[T, R](List.nil[T], function(x: T) {
        sum[R](map[T, R](List.nil[T].remove_one(x), function(y: T) { f(x, y) }))
    }) = List.nil[R]
    map[T, R](List.nil[T], function(x: T) {
        sum[R](map[T, R](List.nil[T].remove_one(x), function(y: T) { f(y, x) }))
    }) = List.nil[R]
    sum[R](List.nil[R]) = R.0
    p(List.nil[T])
    forall(head: T, tail: List[T]) {
        if p(tail) {
            p(tail) = (tail.is_unique implies (sum[R](map[T, R](tail, function(x: T) {
                sum[R](map[T, R](tail.remove_one(x), function(y: T) { f(x, y) }))
            })) = sum[R](map[T, R](tail, function(x: T) {
                sum[R](map[T, R](tail.remove_one(x), function(y: T) { f(y, x) }))
            }))))
            if List.cons(head, tail).is_unique {
                unique_cons_not_contains_head[T](head, tail)
                not tail.contains(head)
                unique_implies_tail_unique[T](head, tail)
                List.cons(head, tail).is_unique implies tail.is_unique
                tail.is_unique
                // LHS: peel x = head
                sum_map_remove_one[T, R](List.cons(head, tail), head, function(x: T) {
                    sum[R](map[T, R](List.cons(head, tail).remove_one(x), function(y: T) { f(x, y) }))
                })
                List.cons(head, tail).contains(head)
                sum[R](map[T, R](List.cons(head, tail).remove_one(head), function(y: T) { f(head, y) })) +
                    sum[R](map[T, R](tail, function(x: T) {
                        sum[R](map[T, R](List.cons(head, tail).remove_one(x), function(y: T) { f(x, y) }))
                    })) = sum[R](map[T, R](List.cons(head, tail), function(x: T) {
                        sum[R](map[T, R](List.cons(head, tail).remove_one(x), function(y: T) { f(x, y) }))
                    }))
                remove_one_cons_eq(head, tail)
                List.cons(head, tail).remove_one(head) = tail
                sum[R](map[T, R](tail, function(y: T) { f(head, y) })) +
                    sum[R](map[T, R](tail, function(x: T) {
                        sum[R](map[T, R](List.cons(head, tail).remove_one(x), function(y: T) { f(x, y) }))
                    })) = sum[R](map[T, R](List.cons(head, tail), function(x: T) {
                        sum[R](map[T, R](List.cons(head, tail).remove_one(x), function(y: T) { f(x, y) }))
                    }))
                // for x in tail: remove_one(x) of cons(head, tail) = cons(head, tail.remove_one(x)), and peel y = head
                forall(x: T) {
                    if tail.contains(x) {
                        if x = head {
                            tail.contains(x) = tail.contains(head)
                            not tail.contains(head)
                            tail.contains(head) = false
                            tail.contains(x) = false
                            false
                        }
                        x != head
                        remove_one_cons_neq[T](head, tail, x)
                        List.cons(head, tail).remove_one(x) = List.cons(head, tail.remove_one(x))
                        sum_map_remove_one[T, R](List.cons(head, tail.remove_one(x)), head, function(y: T) { f(x, y) })
                        List.cons(head, tail.remove_one(x)).contains(head)
                        f(x, head) + sum[R](map[T, R](List.cons(head, tail.remove_one(x)).remove_one(head), function(y: T) { f(x, y) })) =
                            sum[R](map[T, R](List.cons(head, tail.remove_one(x)), function(y: T) { f(x, y) }))
                        remove_one_cons_eq(head, tail.remove_one(x))
                        List.cons(head, tail.remove_one(x)).remove_one(head) = tail.remove_one(x)
                        f(x, head) + sum[R](map[T, R](tail.remove_one(x), function(y: T) { f(x, y) })) =
                            sum[R](map[T, R](List.cons(head, tail.remove_one(x)), function(y: T) { f(x, y) }))
                        sum[R](map[T, R](List.cons(head, tail).remove_one(x), function(y: T) { f(x, y) })) =
                            f(x, head) + sum[R](map[T, R](tail.remove_one(x), function(y: T) { f(x, y) }))
                    }
                }
                sum_map_of_pointwise[T, R](tail, function(x: T) {
                    sum[R](map[T, R](List.cons(head, tail).remove_one(x), function(y: T) { f(x, y) }))
                }, function(x: T) {
                    f(x, head) + sum[R](map[T, R](tail.remove_one(x), function(y: T) { f(x, y) }))
                })
                sum[R](map[T, R](tail, function(x: T) {
                    sum[R](map[T, R](List.cons(head, tail).remove_one(x), function(y: T) { f(x, y) }))
                })) = sum[R](map[T, R](tail, function(x: T) {
                    f(x, head) + sum[R](map[T, R](tail.remove_one(x), function(y: T) { f(x, y) }))
                }))
                sum_map_add_fn[T, R](tail, function(x: T) { f(x, head) }, function(x: T) {
                    sum[R](map[T, R](tail.remove_one(x), function(y: T) { f(x, y) }))
                })
                sum[R](map[T, R](tail, function(x: T) {
                    f(x, head) + sum[R](map[T, R](tail.remove_one(x), function(y: T) { f(x, y) }))
                })) = sum[R](map[T, R](tail, function(x: T) { f(x, head) })) +
                    sum[R](map[T, R](tail, function(x: T) {
                        sum[R](map[T, R](tail.remove_one(x), function(y: T) { f(x, y) }))
                    }))
                sum[R](map[T, R](tail, function(x: T) {
                    sum[R](map[T, R](List.cons(head, tail).remove_one(x), function(y: T) { f(x, y) }))
                })) = sum[R](map[T, R](tail, function(x: T) { f(x, head) })) +
                    sum[R](map[T, R](tail, function(x: T) {
                        sum[R](map[T, R](tail.remove_one(x), function(y: T) { f(x, y) }))
                    }))
                // LHS assembled
                sum[R](map[T, R](tail, function(y: T) { f(head, y) })) +
                    sum[R](map[T, R](tail, function(x: T) { f(x, head) })) +
                    sum[R](map[T, R](tail, function(x: T) {
                        sum[R](map[T, R](tail.remove_one(x), function(y: T) { f(x, y) }))
                    })) = sum[R](map[T, R](List.cons(head, tail), function(x: T) {
                        sum[R](map[T, R](List.cons(head, tail).remove_one(x), function(y: T) { f(x, y) }))
                    }))
                // RHS: peel x = head
                sum_map_remove_one[T, R](List.cons(head, tail), head, function(x: T) {
                    sum[R](map[T, R](List.cons(head, tail).remove_one(x), function(y: T) { f(y, x) }))
                })
                List.cons(head, tail).contains(head)
                sum[R](map[T, R](List.cons(head, tail).remove_one(head), function(y: T) { f(y, head) })) +
                    sum[R](map[T, R](tail, function(x: T) {
                        sum[R](map[T, R](List.cons(head, tail).remove_one(x), function(y: T) { f(y, x) }))
                    })) = sum[R](map[T, R](List.cons(head, tail), function(x: T) {
                        sum[R](map[T, R](List.cons(head, tail).remove_one(x), function(y: T) { f(y, x) }))
                    }))
                remove_one_cons_eq(head, tail)
                List.cons(head, tail).remove_one(head) = tail
                sum[R](map[T, R](tail, function(y: T) { f(y, head) })) +
                    sum[R](map[T, R](tail, function(x: T) {
                        sum[R](map[T, R](List.cons(head, tail).remove_one(x), function(y: T) { f(y, x) }))
                    })) = sum[R](map[T, R](List.cons(head, tail), function(x: T) {
                        sum[R](map[T, R](List.cons(head, tail).remove_one(x), function(y: T) { f(y, x) }))
                    }))
                // for x in tail: peel y = head on the RHS side
                forall(x: T) {
                    if tail.contains(x) {
                        if x = head {
                            tail.contains(x) = tail.contains(head)
                            not tail.contains(head)
                            tail.contains(head) = false
                            tail.contains(x) = false
                            false
                        }
                        x != head
                        remove_one_cons_neq[T](head, tail, x)
                        List.cons(head, tail).remove_one(x) = List.cons(head, tail.remove_one(x))
                        sum_map_remove_one[T, R](List.cons(head, tail.remove_one(x)), head, function(y: T) { f(y, x) })
                        List.cons(head, tail.remove_one(x)).contains(head)
                        f(head, x) + sum[R](map[T, R](List.cons(head, tail.remove_one(x)).remove_one(head), function(y: T) { f(y, x) })) =
                            sum[R](map[T, R](List.cons(head, tail.remove_one(x)), function(y: T) { f(y, x) }))
                        remove_one_cons_eq(head, tail.remove_one(x))
                        List.cons(head, tail.remove_one(x)).remove_one(head) = tail.remove_one(x)
                        f(head, x) + sum[R](map[T, R](tail.remove_one(x), function(y: T) { f(y, x) })) =
                            sum[R](map[T, R](List.cons(head, tail.remove_one(x)), function(y: T) { f(y, x) }))
                        sum[R](map[T, R](List.cons(head, tail).remove_one(x), function(y: T) { f(y, x) })) =
                            f(head, x) + sum[R](map[T, R](tail.remove_one(x), function(y: T) { f(y, x) }))
                    }
                }
                sum_map_of_pointwise[T, R](tail, function(x: T) {
                    sum[R](map[T, R](List.cons(head, tail).remove_one(x), function(y: T) { f(y, x) }))
                }, function(x: T) {
                    f(head, x) + sum[R](map[T, R](tail.remove_one(x), function(y: T) { f(y, x) }))
                })
                sum[R](map[T, R](tail, function(x: T) {
                    sum[R](map[T, R](List.cons(head, tail).remove_one(x), function(y: T) { f(y, x) }))
                })) = sum[R](map[T, R](tail, function(x: T) {
                    f(head, x) + sum[R](map[T, R](tail.remove_one(x), function(y: T) { f(y, x) }))
                }))
                sum_map_add_fn[T, R](tail, function(x: T) { f(head, x) }, function(x: T) {
                    sum[R](map[T, R](tail.remove_one(x), function(y: T) { f(y, x) }))
                })
                sum[R](map[T, R](tail, function(x: T) {
                    f(head, x) + sum[R](map[T, R](tail.remove_one(x), function(y: T) { f(y, x) }))
                })) = sum[R](map[T, R](tail, function(x: T) { f(head, x) })) +
                    sum[R](map[T, R](tail, function(x: T) {
                        sum[R](map[T, R](tail.remove_one(x), function(y: T) { f(y, x) }))
                    }))
                sum[R](map[T, R](tail, function(x: T) {
                    sum[R](map[T, R](List.cons(head, tail).remove_one(x), function(y: T) { f(y, x) }))
                })) = sum[R](map[T, R](tail, function(x: T) { f(head, x) })) +
                    sum[R](map[T, R](tail, function(x: T) {
                        sum[R](map[T, R](tail.remove_one(x), function(y: T) { f(y, x) }))
                    }))
                // RHS assembled
                sum[R](map[T, R](tail, function(y: T) { f(y, head) })) +
                    sum[R](map[T, R](tail, function(x: T) { f(head, x) })) +
                    sum[R](map[T, R](tail, function(x: T) {
                        sum[R](map[T, R](tail.remove_one(x), function(y: T) { f(y, x) }))
                    })) = sum[R](map[T, R](List.cons(head, tail), function(x: T) {
                        sum[R](map[T, R](List.cons(head, tail).remove_one(x), function(y: T) { f(y, x) }))
                    }))
                // renames and IH
                sum[R](map[T, R](tail, function(y: T) { f(head, y) })) =
                    sum[R](map[T, R](tail, function(x: T) { f(head, x) }))
                sum[R](map[T, R](tail, function(x: T) { f(x, head) })) =
                    sum[R](map[T, R](tail, function(y: T) { f(y, head) }))
                p(tail)
                tail.is_unique
                sum[R](map[T, R](tail, function(x: T) {
                    sum[R](map[T, R](tail.remove_one(x), function(y: T) { f(x, y) }))
                })) = sum[R](map[T, R](tail, function(x: T) {
                    sum[R](map[T, R](tail.remove_one(x), function(y: T) { f(y, x) }))
                }))
                p(List.cons(head, tail))
            }
        }
    }
    p(List.nil[T]) and forall(head: T, tail: List[T]) { p(tail) implies p(List.cons(head, tail)) }
    List.induction(p)
    forall(l: List[T]) { p(l) }
    p(cols)
    p(cols) = (cols.is_unique implies (sum[R](map[T, R](cols, function(x: T) {
        sum[R](map[T, R](cols.remove_one(x), function(y: T) { f(x, y) }))
    })) = sum[R](map[T, R](cols, function(x: T) {
        sum[R](map[T, R](cols.remove_one(x), function(y: T) { f(y, x) }))
    }))))
    if cols.is_unique {
        sum[R](map[T, R](cols, function(x: T) {
            sum[R](map[T, R](cols.remove_one(x), function(y: T) { f(x, y) }))
        })) = sum[R](map[T, R](cols, function(x: T) {
            sum[R](map[T, R](cols.remove_one(x), function(y: T) { f(y, x) }))
        }))
    }
}

/// A mapped sum of negations is the negation of the mapped sum.
theorem sum_map_neg[T, R: Ring](items: List[T], f: T -> R) {
    sum[R](map[T, R](items, function(x: T) { -(f(x)) })) = -(sum[R](map[T, R](items, f)))
} by {
    sum_map_scalar_mul[T, R](items, -R.1, f)
    (-R.1) * sum[R](map[T, R](items, f)) = sum[R](map[T, R](items, function(x: T) { (-R.1) * f(x) }))
    mul_neg_one_left[R](sum[R](map[T, R](items, f)))
    (-R.1) * sum[R](map[T, R](items, f)) = -(sum[R](map[T, R](items, f)))
    forall(x: T) {
        if items.contains(x) {
            mul_neg_one_left[R](f(x))
            (-R.1) * f(x) = -(f(x))
            -(f(x)) = (-R.1) * f(x)
        }
    }
    sum_map_of_pointwise[T, R](items, function(x: T) { (-R.1) * f(x) }, function(x: T) { -(f(x)) })
    sum[R](map[T, R](items, function(x: T) { (-R.1) * f(x) })) = sum[R](map[T, R](items, function(x: T) { -(f(x)) }))
    (-R.1) * sum[R](map[T, R](items, f)) = sum[R](map[T, R](items, function(x: T) { -(f(x)) }))
    -(sum[R](map[T, R](items, f))) = sum[R](map[T, R](items, function(x: T) { -(f(x)) }))
    sum[R](map[T, R](items, function(x: T) { -(f(x)) })) = -(sum[R](map[T, R](items, f)))
}

/// The signs of the two summands paired by a row swap are negatives of each
/// other: for distinct columns x, y of a unique column list,
/// alt(idx(x)) * alt(idx_{rem x}(y)) = -(alt(idx(y)) * alt(idx_{rem y}(x))).
theorem det_swap_sign_pair[R: Ring, I](cols: List[I], x: I, y: I) {
    cols.is_unique and cols.contains(x) and cols.contains(y) and x != y implies
    alternating_sign[R](list_index_or_zero[I](cols, x)) *
        alternating_sign[R](list_index_or_zero[I](cols.remove_one(x), y)) =
    -(alternating_sign[R](list_index_or_zero[I](cols, y)) *
        alternating_sign[R](list_index_or_zero[I](cols.remove_one(y), x)))
} by {
    if cols.is_unique and cols.contains(x) and cols.contains(y) and x != y {
        trichotomy(list_index_or_zero[I](cols, x), list_index_or_zero[I](cols, y))
        // case 1: idx(x) < idx(y): removing x shifts y down, removing y leaves x fixed
        if list_index_or_zero[I](cols, x) < list_index_or_zero[I](cols, y) {
            list_index_or_zero_remove_before[I](cols, x, y)
            list_index_or_zero[I](cols.remove_one(x), y).suc = list_index_or_zero[I](cols, y)
            list_index_or_zero_remove_after[I](cols, y, x)
            list_index_or_zero[I](cols.remove_one(y), x) = list_index_or_zero[I](cols, x)
            // LHS sign = alt(a) * alt(b') = alt(b') * alt(a)
            alternating_sign_mul_comm[R](list_index_or_zero[I](cols, x), list_index_or_zero[I](cols.remove_one(x), y))
            alternating_sign[R](list_index_or_zero[I](cols, x)) * alternating_sign[R](list_index_or_zero[I](cols.remove_one(x), y)) =
                alternating_sign[R](list_index_or_zero[I](cols.remove_one(x), y)) * alternating_sign[R](list_index_or_zero[I](cols, x))
            // RHS sign = alt(b) * alt(a) = alt(b'.suc) * alt(a) = (-alt(b')) * alt(a) = -(alt(b') * alt(a))
            alternating_sign_suc[R](list_index_or_zero[I](cols.remove_one(x), y))
            alternating_sign[R](list_index_or_zero[I](cols.remove_one(x), y).suc) = -alternating_sign[R](list_index_or_zero[I](cols.remove_one(x), y))
            alternating_sign[R](list_index_or_zero[I](cols, y)) = -alternating_sign[R](list_index_or_zero[I](cols.remove_one(x), y))
            alternating_sign[R](list_index_or_zero[I](cols, y)) * alternating_sign[R](list_index_or_zero[I](cols.remove_one(y), x)) =
                (-alternating_sign[R](list_index_or_zero[I](cols.remove_one(x), y))) * alternating_sign[R](list_index_or_zero[I](cols, x))
            mul_neg_left[R](alternating_sign[R](list_index_or_zero[I](cols.remove_one(x), y)), alternating_sign[R](list_index_or_zero[I](cols, x)))
            (-alternating_sign[R](list_index_or_zero[I](cols.remove_one(x), y))) * alternating_sign[R](list_index_or_zero[I](cols, x)) =
                -(alternating_sign[R](list_index_or_zero[I](cols.remove_one(x), y)) * alternating_sign[R](list_index_or_zero[I](cols, x)))
            alternating_sign[R](list_index_or_zero[I](cols, y)) * alternating_sign[R](list_index_or_zero[I](cols.remove_one(y), x)) =
                -(alternating_sign[R](list_index_or_zero[I](cols.remove_one(x), y)) * alternating_sign[R](list_index_or_zero[I](cols, x)))
            // -(RHS sign) = -(alt(b') * alt(a)) = alt(b') * alt(a) [inverse_inverse]
            inverse_inverse[R](alternating_sign[R](list_index_or_zero[I](cols.remove_one(x), y)) * alternating_sign[R](list_index_or_zero[I](cols, x)))
            -(-(alternating_sign[R](list_index_or_zero[I](cols.remove_one(x), y)) * alternating_sign[R](list_index_or_zero[I](cols, x)))) =
                alternating_sign[R](list_index_or_zero[I](cols.remove_one(x), y)) * alternating_sign[R](list_index_or_zero[I](cols, x))
            -(alternating_sign[R](list_index_or_zero[I](cols, y)) * alternating_sign[R](list_index_or_zero[I](cols.remove_one(y), x))) =
                alternating_sign[R](list_index_or_zero[I](cols.remove_one(x), y)) * alternating_sign[R](list_index_or_zero[I](cols, x))
            alternating_sign[R](list_index_or_zero[I](cols, x)) * alternating_sign[R](list_index_or_zero[I](cols.remove_one(x), y)) =
                -(alternating_sign[R](list_index_or_zero[I](cols, y)) * alternating_sign[R](list_index_or_zero[I](cols.remove_one(y), x)))
        }
        // case 2: idx(y) < idx(x): removing y shifts x down, removing x leaves y fixed
        if list_index_or_zero[I](cols, y) < list_index_or_zero[I](cols, x) {
            list_index_or_zero_remove_before[I](cols, y, x)
            list_index_or_zero[I](cols.remove_one(y), x).suc = list_index_or_zero[I](cols, x)
            list_index_or_zero_remove_after[I](cols, x, y)
            list_index_or_zero[I](cols.remove_one(x), y) = list_index_or_zero[I](cols, y)
            // RHS sign = alt(b) * alt(a') = alt(a') * alt(b)
            alternating_sign_mul_comm[R](list_index_or_zero[I](cols, y), list_index_or_zero[I](cols.remove_one(y), x))
            alternating_sign[R](list_index_or_zero[I](cols, y)) * alternating_sign[R](list_index_or_zero[I](cols.remove_one(y), x)) =
                alternating_sign[R](list_index_or_zero[I](cols.remove_one(y), x)) * alternating_sign[R](list_index_or_zero[I](cols, y))
            // LHS sign = alt(a) * alt(b) = alt(a'.suc) * alt(b) = (-alt(a')) * alt(b) = -(alt(a') * alt(b))
            alternating_sign_suc[R](list_index_or_zero[I](cols.remove_one(y), x))
            alternating_sign[R](list_index_or_zero[I](cols.remove_one(y), x).suc) = -alternating_sign[R](list_index_or_zero[I](cols.remove_one(y), x))
            alternating_sign[R](list_index_or_zero[I](cols, x)) = -alternating_sign[R](list_index_or_zero[I](cols.remove_one(y), x))
            alternating_sign[R](list_index_or_zero[I](cols, x)) * alternating_sign[R](list_index_or_zero[I](cols.remove_one(x), y)) =
                (-alternating_sign[R](list_index_or_zero[I](cols.remove_one(y), x))) * alternating_sign[R](list_index_or_zero[I](cols, y))
            mul_neg_left[R](alternating_sign[R](list_index_or_zero[I](cols.remove_one(y), x)), alternating_sign[R](list_index_or_zero[I](cols, y)))
            (-alternating_sign[R](list_index_or_zero[I](cols.remove_one(y), x))) * alternating_sign[R](list_index_or_zero[I](cols, y)) =
                -(alternating_sign[R](list_index_or_zero[I](cols.remove_one(y), x)) * alternating_sign[R](list_index_or_zero[I](cols, y)))
            alternating_sign[R](list_index_or_zero[I](cols, x)) * alternating_sign[R](list_index_or_zero[I](cols.remove_one(x), y)) =
                -(alternating_sign[R](list_index_or_zero[I](cols.remove_one(y), x)) * alternating_sign[R](list_index_or_zero[I](cols, y)))
            // -(RHS sign) = -(alt(b) * alt(a')) = -(alt(a') * alt(b)) = LHS sign
            -(alternating_sign[R](list_index_or_zero[I](cols, y)) * alternating_sign[R](list_index_or_zero[I](cols.remove_one(y), x))) =
                -(alternating_sign[R](list_index_or_zero[I](cols.remove_one(y), x)) * alternating_sign[R](list_index_or_zero[I](cols, y)))
            alternating_sign[R](list_index_or_zero[I](cols, x)) * alternating_sign[R](list_index_or_zero[I](cols.remove_one(x), y)) =
                -(alternating_sign[R](list_index_or_zero[I](cols, y)) * alternating_sign[R](list_index_or_zero[I](cols.remove_one(y), x)))
        }
        // case 3: idx(x) = idx(y) contradicts injectivity on a unique list
        if list_index_or_zero[I](cols, x) = list_index_or_zero[I](cols, y) {
            list_index_or_zero_injective_unique[I](cols, x, y)
            x = y
            x != y
            false
        }
        alternating_sign[R](list_index_or_zero[I](cols, x)) *
            alternating_sign[R](list_index_or_zero[I](cols.remove_one(x), y)) =
        -(alternating_sign[R](list_index_or_zero[I](cols, y)) *
            alternating_sign[R](list_index_or_zero[I](cols.remove_one(y), x)))
    }
}

/// One summand of the two-row determinant expansion: the product of the two
/// head-row entries with their column signs and the remaining determinant.
define det_two_row_term[R: Ring, I](entry: (I, I) -> R, r1: I, r2: I, rs: List[I], cols: List[I], x: I, y: I) -> R {
    entry(r1, x) * alternating_sign[R](list_index_or_zero[I](cols, x)) *
        entry(r2, y) * alternating_sign[R](list_index_or_zero[I](cols.remove_one(x), y)) *
        determinant_list[R, I](entry, rs, cols.remove_one(x).remove_one(y))
}

/// The determinant of a matrix with two head rows expands as a double sum over
/// pairs of distinct columns.
theorem determinant_list_two_rows[R: Ring, I](entry: (I, I) -> R, r1: I, r2: I, rs: List[I], cols: List[I]) {
    determinant_list[R, I](entry, List.cons(r1, List.cons(r2, rs)), cols) =
        sum[R](map[I, R](cols, function(x: I) {
            sum[R](map[I, R](cols.remove_one(x), function(y: I) {
                det_two_row_term[R, I](entry, r1, r2, rs, cols, x, y)
            }))
        }))
} by {
    determinant_list_cons_rows_term[R, I](entry, r1, List.cons(r2, rs), cols)
    determinant_list[R, I](entry, List.cons(r1, List.cons(r2, rs)), cols) =
        sum[R](map[I, R](cols, det_row_sum_term[R, I](entry, r1, List.cons(r2, rs), cols)))
    forall(x: I) {
        if cols.contains(x) {
            det_row_sum_term[R, I](entry, r1, List.cons(r2, rs), cols, x) =
                entry(r1, x) * alternating_sign[R](list_index_or_zero[I](cols, x)) *
                    determinant_list[R, I](entry, List.cons(r2, rs), cols.remove_one(x))
            determinant_list_cons_rows_term[R, I](entry, r2, rs, cols.remove_one(x))
            determinant_list[R, I](entry, List.cons(r2, rs), cols.remove_one(x)) =
                sum[R](map[I, R](cols.remove_one(x), det_row_sum_term[R, I](entry, r2, rs, cols.remove_one(x))))
            det_row_sum_term[R, I](entry, r1, List.cons(r2, rs), cols, x) =
                entry(r1, x) * alternating_sign[R](list_index_or_zero[I](cols, x)) *
                    sum[R](map[I, R](cols.remove_one(x), det_row_sum_term[R, I](entry, r2, rs, cols.remove_one(x))))
        }
    }
    sum_map_of_pointwise[I, R](cols, det_row_sum_term[R, I](entry, r1, List.cons(r2, rs), cols),
        function(x: I) {
            entry(r1, x) * alternating_sign[R](list_index_or_zero[I](cols, x)) *
                sum[R](map[I, R](cols.remove_one(x), det_row_sum_term[R, I](entry, r2, rs, cols.remove_one(x))))
        })
    sum[R](map[I, R](cols, det_row_sum_term[R, I](entry, r1, List.cons(r2, rs), cols))) =
        sum[R](map[I, R](cols, function(x: I) {
            entry(r1, x) * alternating_sign[R](list_index_or_zero[I](cols, x)) *
                sum[R](map[I, R](cols.remove_one(x), det_row_sum_term[R, I](entry, r2, rs, cols.remove_one(x))))
        }))
    determinant_list[R, I](entry, List.cons(r1, List.cons(r2, rs)), cols) =
        sum[R](map[I, R](cols, function(x: I) {
            entry(r1, x) * alternating_sign[R](list_index_or_zero[I](cols, x)) *
                sum[R](map[I, R](cols.remove_one(x), det_row_sum_term[R, I](entry, r2, rs, cols.remove_one(x))))
        }))
    forall(x: I) {
        if cols.contains(x) {
            sum_map_scalar_mul[I, R](cols.remove_one(x),
                entry(r1, x) * alternating_sign[R](list_index_or_zero[I](cols, x)),
                det_row_sum_term[R, I](entry, r2, rs, cols.remove_one(x)))
            (entry(r1, x) * alternating_sign[R](list_index_or_zero[I](cols, x))) *
                sum[R](map[I, R](cols.remove_one(x), det_row_sum_term[R, I](entry, r2, rs, cols.remove_one(x)))) =
                sum[R](map[I, R](cols.remove_one(x), function(y: I) {
                    (entry(r1, x) * alternating_sign[R](list_index_or_zero[I](cols, x))) *
                        det_row_sum_term[R, I](entry, r2, rs, cols.remove_one(x), y)
                }))
            entry(r1, x) * alternating_sign[R](list_index_or_zero[I](cols, x)) *
                sum[R](map[I, R](cols.remove_one(x), det_row_sum_term[R, I](entry, r2, rs, cols.remove_one(x)))) =
                sum[R](map[I, R](cols.remove_one(x), function(y: I) {
                    (entry(r1, x) * alternating_sign[R](list_index_or_zero[I](cols, x))) *
                        det_row_sum_term[R, I](entry, r2, rs, cols.remove_one(x), y)
                }))
        }
    }
    sum_map_of_pointwise[I, R](cols, function(x: I) {
        entry(r1, x) * alternating_sign[R](list_index_or_zero[I](cols, x)) *
            sum[R](map[I, R](cols.remove_one(x), det_row_sum_term[R, I](entry, r2, rs, cols.remove_one(x))))
    }, function(x: I) {
        sum[R](map[I, R](cols.remove_one(x), function(y: I) {
            (entry(r1, x) * alternating_sign[R](list_index_or_zero[I](cols, x))) *
                det_row_sum_term[R, I](entry, r2, rs, cols.remove_one(x), y)
        }))
    })
    sum[R](map[I, R](cols, function(x: I) {
        entry(r1, x) * alternating_sign[R](list_index_or_zero[I](cols, x)) *
            sum[R](map[I, R](cols.remove_one(x), det_row_sum_term[R, I](entry, r2, rs, cols.remove_one(x))))
    })) = sum[R](map[I, R](cols, function(x: I) {
        sum[R](map[I, R](cols.remove_one(x), function(y: I) {
            (entry(r1, x) * alternating_sign[R](list_index_or_zero[I](cols, x))) *
                det_row_sum_term[R, I](entry, r2, rs, cols.remove_one(x), y)
        }))
    }))
    determinant_list[R, I](entry, List.cons(r1, List.cons(r2, rs)), cols) =
        sum[R](map[I, R](cols, function(x: I) {
            sum[R](map[I, R](cols.remove_one(x), function(y: I) {
                (entry(r1, x) * alternating_sign[R](list_index_or_zero[I](cols, x))) *
                    det_row_sum_term[R, I](entry, r2, rs, cols.remove_one(x), y)
            }))
        }))
    forall(x: I) {
        if cols.contains(x) {
            forall(y: I) {
                if cols.remove_one(x).contains(y) {
                    det_row_sum_term[R, I](entry, r2, rs, cols.remove_one(x), y) =
                        entry(r2, y) * alternating_sign[R](list_index_or_zero[I](cols.remove_one(x), y)) *
                            determinant_list[R, I](entry, rs, cols.remove_one(x).remove_one(y))
                    (entry(r1, x) * alternating_sign[R](list_index_or_zero[I](cols, x))) *
                        det_row_sum_term[R, I](entry, r2, rs, cols.remove_one(x), y) =
                        (entry(r1, x) * alternating_sign[R](list_index_or_zero[I](cols, x))) *
                            (entry(r2, y) * alternating_sign[R](list_index_or_zero[I](cols.remove_one(x), y)) *
                                determinant_list[R, I](entry, rs, cols.remove_one(x).remove_one(y)))
                    (entry(r1, x) * alternating_sign[R](list_index_or_zero[I](cols, x))) *
                        (entry(r2, y) * alternating_sign[R](list_index_or_zero[I](cols.remove_one(x), y)) *
                            determinant_list[R, I](entry, rs, cols.remove_one(x).remove_one(y))) =
                        entry(r1, x) * alternating_sign[R](list_index_or_zero[I](cols, x)) *
                            entry(r2, y) * alternating_sign[R](list_index_or_zero[I](cols.remove_one(x), y)) *
                            determinant_list[R, I](entry, rs, cols.remove_one(x).remove_one(y))
                    det_two_row_term[R, I](entry, r1, r2, rs, cols, x, y) =
                        entry(r1, x) * alternating_sign[R](list_index_or_zero[I](cols, x)) *
                            entry(r2, y) * alternating_sign[R](list_index_or_zero[I](cols.remove_one(x), y)) *
                            determinant_list[R, I](entry, rs, cols.remove_one(x).remove_one(y))
                    (entry(r1, x) * alternating_sign[R](list_index_or_zero[I](cols, x))) *
                        det_row_sum_term[R, I](entry, r2, rs, cols.remove_one(x), y) =
                        det_two_row_term[R, I](entry, r1, r2, rs, cols, x, y)
                }
            }
            sum_map_of_pointwise[I, R](cols.remove_one(x), function(y: I) {
                (entry(r1, x) * alternating_sign[R](list_index_or_zero[I](cols, x))) *
                    det_row_sum_term[R, I](entry, r2, rs, cols.remove_one(x), y)
            }, function(y: I) {
                det_two_row_term[R, I](entry, r1, r2, rs, cols, x, y)
            })
            sum[R](map[I, R](cols.remove_one(x), function(y: I) {
                (entry(r1, x) * alternating_sign[R](list_index_or_zero[I](cols, x))) *
                    det_row_sum_term[R, I](entry, r2, rs, cols.remove_one(x), y)
            })) = sum[R](map[I, R](cols.remove_one(x), function(y: I) {
                det_two_row_term[R, I](entry, r1, r2, rs, cols, x, y)
            }))
        }
    }
    sum_map_of_pointwise[I, R](cols, function(x: I) {
        sum[R](map[I, R](cols.remove_one(x), function(y: I) {
            (entry(r1, x) * alternating_sign[R](list_index_or_zero[I](cols, x))) *
                det_row_sum_term[R, I](entry, r2, rs, cols.remove_one(x), y)
        }))
    }, function(x: I) {
        sum[R](map[I, R](cols.remove_one(x), function(y: I) {
            det_two_row_term[R, I](entry, r1, r2, rs, cols, x, y)
        }))
    })
    sum[R](map[I, R](cols, function(x: I) {
        sum[R](map[I, R](cols.remove_one(x), function(y: I) {
            (entry(r1, x) * alternating_sign[R](list_index_or_zero[I](cols, x))) *
                det_row_sum_term[R, I](entry, r2, rs, cols.remove_one(x), y)
        }))
    })) = sum[R](map[I, R](cols, function(x: I) {
        sum[R](map[I, R](cols.remove_one(x), function(y: I) {
            det_two_row_term[R, I](entry, r1, r2, rs, cols, x, y)
        }))
    }))
    determinant_list[R, I](entry, List.cons(r1, List.cons(r2, rs)), cols) =
        sum[R](map[I, R](cols, function(x: I) {
            sum[R](map[I, R](cols.remove_one(x), function(y: I) {
                det_two_row_term[R, I](entry, r1, r2, rs, cols, x, y)
            }))
        }))
}

/// The two-row summands paired by a row swap are negatives of each other:
/// T(r2, r1)(x, y) = -T(r1, r2)(y, x) for distinct columns x, y.
theorem swap_term_pair_neg[R: CommRing, I](entry: (I, I) -> R, r1: I, r2: I, rs: List[I], cols: List[I], x: I, y: I) {
    cols.is_unique and cols.contains(x) and cols.remove_one(x).contains(y) implies
    det_two_row_term[R, I](entry, r2, r1, rs, cols, x, y) =
        -(det_two_row_term[R, I](entry, r1, r2, rs, cols, y, x))
} by {
    if cols.is_unique and cols.contains(x) and cols.remove_one(x).contains(y) {
        // x != y from uniqueness of cols
        if y = x {
            cols.remove_one(x).contains(y) = cols.remove_one(x).contains(x)
            remove_one_unique_not_contains_self[I](cols, x)
            not cols.remove_one(x).contains(x)
            cols.remove_one(x).contains(x) = false
            cols.remove_one(x).contains(y) = false
            false
        }
        x != y
        // y occurs in cols
        remove_one_contains_other[I](cols, x, y)
        cols.contains(y) = cols.remove_one(x).contains(y)
        cols.contains(y)
        // sign pairing
        det_swap_sign_pair[R, I](cols, x, y)
        alternating_sign[R](list_index_or_zero[I](cols, x)) *
            alternating_sign[R](list_index_or_zero[I](cols.remove_one(x), y)) =
            -(alternating_sign[R](list_index_or_zero[I](cols, y)) *
                alternating_sign[R](list_index_or_zero[I](cols.remove_one(y), x)))
        // remaining determinant symmetric
        remove_one_comm[I](cols, x, y)
        cols.remove_one(x).remove_one(y) = cols.remove_one(y).remove_one(x)
        determinant_list[R, I](entry, rs, cols.remove_one(x).remove_one(y)) =
            determinant_list[R, I](entry, rs, cols.remove_one(y).remove_one(x))
        // unfold the two terms
        det_two_row_term[R, I](entry, r2, r1, rs, cols, x, y) =
            entry(r2, x) * alternating_sign[R](list_index_or_zero[I](cols, x)) *
                entry(r1, y) * alternating_sign[R](list_index_or_zero[I](cols.remove_one(x), y)) *
                determinant_list[R, I](entry, rs, cols.remove_one(x).remove_one(y))
        det_two_row_term[R, I](entry, r1, r2, rs, cols, y, x) =
            entry(r1, y) * alternating_sign[R](list_index_or_zero[I](cols, y)) *
                entry(r2, x) * alternating_sign[R](list_index_or_zero[I](cols.remove_one(y), x)) *
                determinant_list[R, I](entry, rs, cols.remove_one(y).remove_one(x))
        // combine: gather the signs, pull out the minus, commute the entries
        entry(r2, x) * alternating_sign[R](list_index_or_zero[I](cols, x)) *
            entry(r1, y) * alternating_sign[R](list_index_or_zero[I](cols.remove_one(x), y)) *
            determinant_list[R, I](entry, rs, cols.remove_one(x).remove_one(y)) =
            entry(r2, x) * (alternating_sign[R](list_index_or_zero[I](cols, x)) *
                alternating_sign[R](list_index_or_zero[I](cols.remove_one(x), y))) *
                entry(r1, y) * determinant_list[R, I](entry, rs, cols.remove_one(x).remove_one(y))
        entry(r2, x) * (alternating_sign[R](list_index_or_zero[I](cols, x)) *
            alternating_sign[R](list_index_or_zero[I](cols.remove_one(x), y))) *
            entry(r1, y) * determinant_list[R, I](entry, rs, cols.remove_one(x).remove_one(y)) =
            entry(r2, x) * (-(alternating_sign[R](list_index_or_zero[I](cols, y)) *
                alternating_sign[R](list_index_or_zero[I](cols.remove_one(y), x)))) *
                entry(r1, y) * determinant_list[R, I](entry, rs, cols.remove_one(x).remove_one(y))
        // pull the minus out of the product chain one factor at a time
        mul_neg_right[R](entry(r2, x), alternating_sign[R](list_index_or_zero[I](cols, y)) *
            alternating_sign[R](list_index_or_zero[I](cols.remove_one(y), x)))
        entry(r2, x) * (-(alternating_sign[R](list_index_or_zero[I](cols, y)) *
            alternating_sign[R](list_index_or_zero[I](cols.remove_one(y), x)))) =
            -(entry(r2, x) * (alternating_sign[R](list_index_or_zero[I](cols, y)) *
                alternating_sign[R](list_index_or_zero[I](cols.remove_one(y), x))))
        mul_neg_left[R](entry(r2, x) * (alternating_sign[R](list_index_or_zero[I](cols, y)) *
            alternating_sign[R](list_index_or_zero[I](cols.remove_one(y), x))), entry(r1, y))
        (-(entry(r2, x) * (alternating_sign[R](list_index_or_zero[I](cols, y)) *
            alternating_sign[R](list_index_or_zero[I](cols.remove_one(y), x))))) * entry(r1, y) =
            -(entry(r2, x) * (alternating_sign[R](list_index_or_zero[I](cols, y)) *
                alternating_sign[R](list_index_or_zero[I](cols.remove_one(y), x))) * entry(r1, y))
        mul_neg_left[R](entry(r2, x) * (alternating_sign[R](list_index_or_zero[I](cols, y)) *
            alternating_sign[R](list_index_or_zero[I](cols.remove_one(y), x))) * entry(r1, y),
            determinant_list[R, I](entry, rs, cols.remove_one(x).remove_one(y)))
        (-(entry(r2, x) * (alternating_sign[R](list_index_or_zero[I](cols, y)) *
            alternating_sign[R](list_index_or_zero[I](cols.remove_one(y), x))) * entry(r1, y))) *
            determinant_list[R, I](entry, rs, cols.remove_one(x).remove_one(y)) =
            -(entry(r2, x) * (alternating_sign[R](list_index_or_zero[I](cols, y)) *
                alternating_sign[R](list_index_or_zero[I](cols.remove_one(y), x))) * entry(r1, y) *
                determinant_list[R, I](entry, rs, cols.remove_one(x).remove_one(y)))
        entry(r2, x) * (-(alternating_sign[R](list_index_or_zero[I](cols, y)) *
            alternating_sign[R](list_index_or_zero[I](cols.remove_one(y), x)))) *
            entry(r1, y) * determinant_list[R, I](entry, rs, cols.remove_one(x).remove_one(y)) =
            -(entry(r2, x) * (alternating_sign[R](list_index_or_zero[I](cols, y)) *
                alternating_sign[R](list_index_or_zero[I](cols.remove_one(y), x))) *
                entry(r1, y) * determinant_list[R, I](entry, rs, cols.remove_one(x).remove_one(y)))
        -(entry(r2, x) * (alternating_sign[R](list_index_or_zero[I](cols, y)) *
            alternating_sign[R](list_index_or_zero[I](cols.remove_one(y), x))) *
            entry(r1, y) * determinant_list[R, I](entry, rs, cols.remove_one(x).remove_one(y))) =
            -(entry(r1, y) * (alternating_sign[R](list_index_or_zero[I](cols, y)) *
                alternating_sign[R](list_index_or_zero[I](cols.remove_one(y), x))) *
                entry(r2, x) * determinant_list[R, I](entry, rs, cols.remove_one(x).remove_one(y)))
        -(entry(r1, y) * (alternating_sign[R](list_index_or_zero[I](cols, y)) *
            alternating_sign[R](list_index_or_zero[I](cols.remove_one(y), x))) *
            entry(r2, x) * determinant_list[R, I](entry, rs, cols.remove_one(x).remove_one(y))) =
            -(entry(r1, y) * alternating_sign[R](list_index_or_zero[I](cols, y)) *
                entry(r2, x) * alternating_sign[R](list_index_or_zero[I](cols.remove_one(y), x)) *
                determinant_list[R, I](entry, rs, cols.remove_one(x).remove_one(y)))
        -(entry(r1, y) * alternating_sign[R](list_index_or_zero[I](cols, y)) *
            entry(r2, x) * alternating_sign[R](list_index_or_zero[I](cols.remove_one(y), x)) *
            determinant_list[R, I](entry, rs, cols.remove_one(x).remove_one(y))) =
            -(entry(r1, y) * alternating_sign[R](list_index_or_zero[I](cols, y)) *
                entry(r2, x) * alternating_sign[R](list_index_or_zero[I](cols.remove_one(y), x)) *
                determinant_list[R, I](entry, rs, cols.remove_one(y).remove_one(x)))
        entry(r2, x) * alternating_sign[R](list_index_or_zero[I](cols, x)) *
            entry(r1, y) * alternating_sign[R](list_index_or_zero[I](cols.remove_one(x), y)) *
            determinant_list[R, I](entry, rs, cols.remove_one(x).remove_one(y)) =
            -(entry(r1, y) * alternating_sign[R](list_index_or_zero[I](cols, y)) *
                entry(r2, x) * alternating_sign[R](list_index_or_zero[I](cols.remove_one(y), x)) *
                determinant_list[R, I](entry, rs, cols.remove_one(y).remove_one(x)))
        det_two_row_term[R, I](entry, r2, r1, rs, cols, x, y) =
            -(det_two_row_term[R, I](entry, r1, r2, rs, cols, y, x))
    }
}

/// Swapping the first two rows negates the determinant.
theorem determinant_list_swap_first_two_rows[R: CommRing, I](entry: (I, I) -> R, r1: I, r2: I, rs: List[I], cols: List[I]) {
    cols.is_unique implies
    determinant_list[R, I](entry, List.cons(r2, List.cons(r1, rs)), cols) =
        -(determinant_list[R, I](entry, List.cons(r1, List.cons(r2, rs)), cols))
} by {
    if cols.is_unique {
        // LHS expansion along the two head rows
        determinant_list_two_rows[R, I](entry, r2, r1, rs, cols)
        determinant_list[R, I](entry, List.cons(r2, List.cons(r1, rs)), cols) =
            sum[R](map[I, R](cols, function(x: I) {
                sum[R](map[I, R](cols.remove_one(x), function(y: I) {
                    det_two_row_term[R, I](entry, r2, r1, rs, cols, x, y)
                }))
            }))
        // RHS expansion along the two head rows
        determinant_list_two_rows[R, I](entry, r1, r2, rs, cols)
        determinant_list[R, I](entry, List.cons(r1, List.cons(r2, rs)), cols) =
            sum[R](map[I, R](cols, function(x: I) {
                sum[R](map[I, R](cols.remove_one(x), function(y: I) {
                    det_two_row_term[R, I](entry, r1, r2, rs, cols, x, y)
                }))
            }))
        // pair the LHS summand at (x, y) with the RHS summand at (y, x)
        forall(x: I) {
            if cols.contains(x) {
                forall(y: I) {
                    if cols.remove_one(x).contains(y) {
                        swap_term_pair_neg[R, I](entry, r1, r2, rs, cols, x, y)
                        det_two_row_term[R, I](entry, r2, r1, rs, cols, x, y) =
                            -(det_two_row_term[R, I](entry, r1, r2, rs, cols, y, x))
                    }
                }
                sum_map_of_pointwise[I, R](cols.remove_one(x),
                    function(y: I) { det_two_row_term[R, I](entry, r2, r1, rs, cols, x, y) },
                    function(y: I) { -(det_two_row_term[R, I](entry, r1, r2, rs, cols, y, x)) })
                sum[R](map[I, R](cols.remove_one(x), function(y: I) {
                    det_two_row_term[R, I](entry, r2, r1, rs, cols, x, y)
                })) = sum[R](map[I, R](cols.remove_one(x), function(y: I) {
                    -(det_two_row_term[R, I](entry, r1, r2, rs, cols, y, x))
                }))
            }
        }
        sum_map_of_pointwise[I, R](cols,
            function(x: I) {
                sum[R](map[I, R](cols.remove_one(x), function(y: I) {
                    det_two_row_term[R, I](entry, r2, r1, rs, cols, x, y)
                }))
            },
            function(x: I) {
                sum[R](map[I, R](cols.remove_one(x), function(y: I) {
                    -(det_two_row_term[R, I](entry, r1, r2, rs, cols, y, x))
                }))
            })
        sum[R](map[I, R](cols, function(x: I) {
            sum[R](map[I, R](cols.remove_one(x), function(y: I) {
                det_two_row_term[R, I](entry, r2, r1, rs, cols, x, y)
            }))
        })) = sum[R](map[I, R](cols, function(x: I) {
            sum[R](map[I, R](cols.remove_one(x), function(y: I) {
                -(det_two_row_term[R, I](entry, r1, r2, rs, cols, y, x))
            }))
        }))
        determinant_list[R, I](entry, List.cons(r2, List.cons(r1, rs)), cols) =
            sum[R](map[I, R](cols, function(x: I) {
                sum[R](map[I, R](cols.remove_one(x), function(y: I) {
                    -(det_two_row_term[R, I](entry, r1, r2, rs, cols, y, x))
                }))
            }))
        // pull the negation out of the inner sums
        forall(x: I) {
            if cols.contains(x) {
                sum_map_neg[I, R](cols.remove_one(x),
                    function(y: I) { det_two_row_term[R, I](entry, r1, r2, rs, cols, y, x) })
                sum[R](map[I, R](cols.remove_one(x), function(y: I) {
                    -(det_two_row_term[R, I](entry, r1, r2, rs, cols, y, x))
                })) = -(sum[R](map[I, R](cols.remove_one(x), function(y: I) {
                    det_two_row_term[R, I](entry, r1, r2, rs, cols, y, x)
                })))
            }
        }
        sum_map_of_pointwise[I, R](cols,
            function(x: I) {
                sum[R](map[I, R](cols.remove_one(x), function(y: I) {
                    -(det_two_row_term[R, I](entry, r1, r2, rs, cols, y, x))
                }))
            },
            function(x: I) {
                -(sum[R](map[I, R](cols.remove_one(x), function(y: I) {
                    det_two_row_term[R, I](entry, r1, r2, rs, cols, y, x)
                })))
            })
        sum[R](map[I, R](cols, function(x: I) {
            sum[R](map[I, R](cols.remove_one(x), function(y: I) {
                -(det_two_row_term[R, I](entry, r1, r2, rs, cols, y, x))
            }))
        })) = sum[R](map[I, R](cols, function(x: I) {
            -(sum[R](map[I, R](cols.remove_one(x), function(y: I) {
                det_two_row_term[R, I](entry, r1, r2, rs, cols, y, x)
            })))
        }))
        determinant_list[R, I](entry, List.cons(r2, List.cons(r1, rs)), cols) =
            sum[R](map[I, R](cols, function(x: I) {
                -(sum[R](map[I, R](cols.remove_one(x), function(y: I) {
                    det_two_row_term[R, I](entry, r1, r2, rs, cols, y, x)
                })))
            }))
        // pull the negation out of the outer sum
        sum_map_neg[I, R](cols, function(x: I) {
            sum[R](map[I, R](cols.remove_one(x), function(y: I) {
                det_two_row_term[R, I](entry, r1, r2, rs, cols, y, x)
            }))
        })
        sum[R](map[I, R](cols, function(x: I) {
            -(sum[R](map[I, R](cols.remove_one(x), function(y: I) {
                det_two_row_term[R, I](entry, r1, r2, rs, cols, y, x)
            })))
        })) = -(sum[R](map[I, R](cols, function(x: I) {
            sum[R](map[I, R](cols.remove_one(x), function(y: I) {
                det_two_row_term[R, I](entry, r1, r2, rs, cols, y, x)
            }))
        })))
        determinant_list[R, I](entry, List.cons(r2, List.cons(r1, rs)), cols) =
            -(sum[R](map[I, R](cols, function(x: I) {
                sum[R](map[I, R](cols.remove_one(x), function(y: I) {
                    det_two_row_term[R, I](entry, r1, r2, rs, cols, y, x)
                }))
            })))
        // reindex the double sum: (x, y) -> (y, x)
        sum_double_remove_flip[I, R](cols, function(x: I, y: I) {
            det_two_row_term[R, I](entry, r1, r2, rs, cols, x, y)
        })
        sum[R](map[I, R](cols, function(x: I) {
            sum[R](map[I, R](cols.remove_one(x), function(y: I) {
                det_two_row_term[R, I](entry, r1, r2, rs, cols, x, y)
            }))
        })) = sum[R](map[I, R](cols, function(x: I) {
            sum[R](map[I, R](cols.remove_one(x), function(y: I) {
                det_two_row_term[R, I](entry, r1, r2, rs, cols, y, x)
            }))
        }))
        sum[R](map[I, R](cols, function(x: I) {
            sum[R](map[I, R](cols.remove_one(x), function(y: I) {
                det_two_row_term[R, I](entry, r1, r2, rs, cols, y, x)
            }))
        })) = sum[R](map[I, R](cols, function(x: I) {
            sum[R](map[I, R](cols.remove_one(x), function(y: I) {
                det_two_row_term[R, I](entry, r1, r2, rs, cols, x, y)
            }))
        }))
        determinant_list[R, I](entry, List.cons(r2, List.cons(r1, rs)), cols) =
            -(sum[R](map[I, R](cols, function(x: I) {
                sum[R](map[I, R](cols.remove_one(x), function(y: I) {
                    det_two_row_term[R, I](entry, r1, r2, rs, cols, x, y)
                }))
            })))
        // the inner double sum is the RHS determinant
        -(sum[R](map[I, R](cols, function(x: I) {
            sum[R](map[I, R](cols.remove_one(x), function(y: I) {
                det_two_row_term[R, I](entry, r1, r2, rs, cols, x, y)
            }))
        }))) = -(determinant_list[R, I](entry, List.cons(r1, List.cons(r2, rs)), cols))
        determinant_list[R, I](entry, List.cons(r2, List.cons(r1, rs)), cols) =
            -(determinant_list[R, I](entry, List.cons(r1, List.cons(r2, rs)), cols))
    }
}

/// The entry function obtained by replacing one row with a supplied row function.
define det_row_replace_entry[R: Ring, I](entry: (I, I) -> R, r: I, g: (I) -> R, i: I, c: I) -> R {
    if i = r {
        g(c)
    } else {
        entry(i, c)
    }
}

/// The pointwise sum of two row functions.
define det_row_fn_add[R: Ring, I](g1: (I) -> R, g2: (I) -> R, c: I) -> R {
    g1(c) + g2(c)
}

/// The pointwise scalar multiple of a row function.
define det_row_fn_smul[R: Ring, I](c: R, g: (I) -> R, i: I) -> R {
    c * g(i)
}

/// Replacing a row that is absent from the row list leaves a determinant-list unchanged.
theorem determinant_list_row_absent[R: Ring, I](entry: (I, I) -> R, g: (I) -> R, r: I, rows: List[I], cols: List[I]) {
    not rows.contains(r) implies
        determinant_list[R, I](det_row_replace_entry[R, I](entry, r, g), rows, cols) =
            determinant_list[R, I](entry, rows, cols)
} by {
    if not rows.contains(r) {
        define p(l: List[I]) -> Bool {
            not l.contains(r) implies
                forall(cols2: List[I]) {
                    determinant_list[R, I](det_row_replace_entry[R, I](entry, r, g), l, cols2) =
                        determinant_list[R, I](entry, l, cols2)
                }
        }
        if not List.nil[I].contains(r) {
            determinant_list_nil_rows[R, I](det_row_replace_entry[R, I](entry, r, g), List.nil[I])
            determinant_list[R, I](det_row_replace_entry[R, I](entry, r, g), List.nil[I], List.nil[I]) = R.1
            determinant_list_nil_rows[R, I](entry, List.nil[I])
            determinant_list[R, I](entry, List.nil[I], List.nil[I]) = R.1
            forall(cols2: List[I]) {
                determinant_list[R, I](det_row_replace_entry[R, I](entry, r, g), List.nil[I], cols2) =
                    determinant_list[R, I](entry, List.nil[I], cols2)
            }
        }
        p(List.nil[I])
        forall(head: I, tail: List[I]) {
            if p(tail) {
                if not List.cons(head, tail).contains(r) {
                    if head = r {
                        List.cons(head, tail).contains(r) = true
                        List.cons(head, tail).contains(r) = false
                        false
                    }
                    if head != r {
                        forall(cols2: List[I]) {
                            determinant_list_cons_rows_term[R, I](det_row_replace_entry[R, I](entry, r, g), head, tail, cols2)
                            determinant_list[R, I](det_row_replace_entry[R, I](entry, r, g), List.cons(head, tail), cols2) =
                                sum[R](map[I, R](cols2, det_row_sum_term[R, I](det_row_replace_entry[R, I](entry, r, g), head, tail, cols2)))
                            determinant_list_cons_rows_term[R, I](entry, head, tail, cols2)
                            determinant_list[R, I](entry, List.cons(head, tail), cols2) =
                                sum[R](map[I, R](cols2, det_row_sum_term[R, I](entry, head, tail, cols2)))
                            List.cons(head, tail).contains(r) = tail.contains(r)
                            not tail.contains(r)
                            p(tail) = (not tail.contains(r) implies
                                forall(cols3: List[I]) {
                                    determinant_list[R, I](det_row_replace_entry[R, I](entry, r, g), tail, cols3) =
                                        determinant_list[R, I](entry, tail, cols3)
                                })
                            forall(cols3: List[I]) {
                                determinant_list[R, I](det_row_replace_entry[R, I](entry, r, g), tail, cols3) =
                                    determinant_list[R, I](entry, tail, cols3)
                            }
                            forall(col: I) {
                                if cols2.contains(col) {
                                    det_row_replace_entry[R, I](entry, r, g, head, col) = entry(head, col)
                                    determinant_list[R, I](det_row_replace_entry[R, I](entry, r, g), tail, cols2.remove_one(col)) =
                                        determinant_list[R, I](entry, tail, cols2.remove_one(col))
                                    det_row_sum_term[R, I](det_row_replace_entry[R, I](entry, r, g), head, tail, cols2, col) =
                                        entry(head, col) * alternating_sign[R](list_index_or_zero[I](cols2, col)) *
                                            determinant_list[R, I](det_row_replace_entry[R, I](entry, r, g), tail, cols2.remove_one(col))
                                    det_row_sum_term[R, I](entry, head, tail, cols2, col) =
                                        entry(head, col) * alternating_sign[R](list_index_or_zero[I](cols2, col)) *
                                            determinant_list[R, I](entry, tail, cols2.remove_one(col))
                                    det_row_sum_term[R, I](det_row_replace_entry[R, I](entry, r, g), head, tail, cols2, col) =
                                        det_row_sum_term[R, I](entry, head, tail, cols2, col)
                                }
                            }
                            sum_map_of_pointwise[I, R](cols2,
                                det_row_sum_term[R, I](det_row_replace_entry[R, I](entry, r, g), head, tail, cols2),
                                det_row_sum_term[R, I](entry, head, tail, cols2))
                            sum[R](map[I, R](cols2, det_row_sum_term[R, I](det_row_replace_entry[R, I](entry, r, g), head, tail, cols2))) =
                                sum[R](map[I, R](cols2, det_row_sum_term[R, I](entry, head, tail, cols2)))
                            determinant_list[R, I](det_row_replace_entry[R, I](entry, r, g), List.cons(head, tail), cols2) =
                                determinant_list[R, I](entry, List.cons(head, tail), cols2)
                        }
                    }
                    forall(cols2: List[I]) {
                        determinant_list[R, I](det_row_replace_entry[R, I](entry, r, g), List.cons(head, tail), cols2) =
                            determinant_list[R, I](entry, List.cons(head, tail), cols2)
                    }
                }
                p(tail) implies p(List.cons(head, tail))
            }
        }
        p(List.nil[I]) and forall(head: I, tail: List[I]) { p(tail) implies p(List.cons(head, tail)) }
        List.induction(p)
        forall(l: List[I]) { p(l) }
        p(rows)
        forall(cols2: List[I]) {
            determinant_list[R, I](det_row_replace_entry[R, I](entry, r, g), rows, cols2) =
                determinant_list[R, I](entry, rows, cols2)
        }
        determinant_list[R, I](det_row_replace_entry[R, I](entry, r, g), rows, cols) =
            determinant_list[R, I](entry, rows, cols)
    }
}

/// A determinant-list is additive in a single row: replacing the row `r` of an
/// entry function by the sum of two row functions splits the determinant into
/// the sum of the two determinants, provided the row `r` occurs exactly once in
/// the row list.
theorem determinant_list_row_add[R: Ring, I](entry: (I, I) -> R, g1: (I) -> R, g2: (I) -> R, r: I, rows: List[I], cols: List[I]) {
    rows.contains(r) and not rows.remove_one(r).contains(r) implies
        determinant_list[R, I](det_row_replace_entry[R, I](entry, r, det_row_fn_add[R, I](g1, g2)), rows, cols) =
            determinant_list[R, I](det_row_replace_entry[R, I](entry, r, g1), rows, cols) +
                determinant_list[R, I](det_row_replace_entry[R, I](entry, r, g2), rows, cols)
} by {
    if rows.contains(r) and not rows.remove_one(r).contains(r) {
        define p(l: List[I]) -> Bool {
            l.contains(r) and not l.remove_one(r).contains(r) implies
                forall(cols2: List[I]) {
                    determinant_list[R, I](det_row_replace_entry[R, I](entry, r, det_row_fn_add[R, I](g1, g2)), l, cols2) =
                        determinant_list[R, I](det_row_replace_entry[R, I](entry, r, g1), l, cols2) +
                            determinant_list[R, I](det_row_replace_entry[R, I](entry, r, g2), l, cols2)
                }
        }
        if List.nil[I].contains(r) and not List.nil[I].remove_one(r).contains(r) {
            false
        }
        p(List.nil[I])
        forall(head: I, tail: List[I]) {
            if p(tail) {
                if List.cons(head, tail).contains(r) and not List.cons(head, tail).remove_one(r).contains(r) {
                    if head = r {
                        remove_one_cons_eq(head, tail)
                        List.cons(head, tail).remove_one(r) = tail
                        not tail.contains(r)
                        forall(cols2: List[I]) {
                            determinant_list_row_absent[R, I](entry, det_row_fn_add[R, I](g1, g2), r, tail, cols2)
                            determinant_list[R, I](det_row_replace_entry[R, I](entry, r, det_row_fn_add[R, I](g1, g2)), tail, cols2) =
                                determinant_list[R, I](entry, tail, cols2)
                            determinant_list_row_absent[R, I](entry, g1, r, tail, cols2)
                            determinant_list[R, I](det_row_replace_entry[R, I](entry, r, g1), tail, cols2) =
                                determinant_list[R, I](entry, tail, cols2)
                            determinant_list_row_absent[R, I](entry, g2, r, tail, cols2)
                            determinant_list[R, I](det_row_replace_entry[R, I](entry, r, g2), tail, cols2) =
                                determinant_list[R, I](entry, tail, cols2)
                            determinant_list[R, I](det_row_replace_entry[R, I](entry, r, det_row_fn_add[R, I](g1, g2)), tail, cols2) =
                                determinant_list[R, I](det_row_replace_entry[R, I](entry, r, g1), tail, cols2)
                            determinant_list[R, I](det_row_replace_entry[R, I](entry, r, det_row_fn_add[R, I](g1, g2)), tail, cols2) =
                                determinant_list[R, I](det_row_replace_entry[R, I](entry, r, g2), tail, cols2)
                            determinant_list_cons_rows_term[R, I](det_row_replace_entry[R, I](entry, r, det_row_fn_add[R, I](g1, g2)), head, tail, cols2)
                            determinant_list[R, I](det_row_replace_entry[R, I](entry, r, det_row_fn_add[R, I](g1, g2)), List.cons(head, tail), cols2) =
                                sum[R](map[I, R](cols2, det_row_sum_term[R, I](det_row_replace_entry[R, I](entry, r, det_row_fn_add[R, I](g1, g2)), head, tail, cols2)))
                            determinant_list_cons_rows_term[R, I](det_row_replace_entry[R, I](entry, r, g1), head, tail, cols2)
                            determinant_list[R, I](det_row_replace_entry[R, I](entry, r, g1), List.cons(head, tail), cols2) =
                                sum[R](map[I, R](cols2, det_row_sum_term[R, I](det_row_replace_entry[R, I](entry, r, g1), head, tail, cols2)))
                            determinant_list_cons_rows_term[R, I](det_row_replace_entry[R, I](entry, r, g2), head, tail, cols2)
                            determinant_list[R, I](det_row_replace_entry[R, I](entry, r, g2), List.cons(head, tail), cols2) =
                                sum[R](map[I, R](cols2, det_row_sum_term[R, I](det_row_replace_entry[R, I](entry, r, g2), head, tail, cols2)))
                            forall(col: I) {
                                if cols2.contains(col) {
                                    det_row_sum_term[R, I](det_row_replace_entry[R, I](entry, r, det_row_fn_add[R, I](g1, g2)), head, tail, cols2, col) =
                                        det_row_replace_entry[R, I](entry, r, det_row_fn_add[R, I](g1, g2), head, col) *
                                            alternating_sign[R](list_index_or_zero[I](cols2, col)) *
                                            determinant_list[R, I](det_row_replace_entry[R, I](entry, r, det_row_fn_add[R, I](g1, g2)), tail, cols2.remove_one(col))
                                    det_row_replace_entry[R, I](entry, r, det_row_fn_add[R, I](g1, g2), head, col) = det_row_fn_add[R, I](g1, g2, col)
                                    det_row_fn_add[R, I](g1, g2, col) = g1(col) + g2(col)
                                    det_row_replace_entry[R, I](entry, r, det_row_fn_add[R, I](g1, g2), head, col) = g1(col) + g2(col)
                                    determinant_list_row_absent[R, I](entry, det_row_fn_add[R, I](g1, g2), r, tail, cols2.remove_one(col))
                                    determinant_list[R, I](det_row_replace_entry[R, I](entry, r, det_row_fn_add[R, I](g1, g2)), tail, cols2.remove_one(col)) =
                                        determinant_list[R, I](entry, tail, cols2.remove_one(col))
                                    determinant_list_row_absent[R, I](entry, g1, r, tail, cols2.remove_one(col))
                                    determinant_list[R, I](det_row_replace_entry[R, I](entry, r, g1), tail, cols2.remove_one(col)) =
                                        determinant_list[R, I](entry, tail, cols2.remove_one(col))
                                    determinant_list_row_absent[R, I](entry, g2, r, tail, cols2.remove_one(col))
                                    determinant_list[R, I](det_row_replace_entry[R, I](entry, r, g2), tail, cols2.remove_one(col)) =
                                        determinant_list[R, I](entry, tail, cols2.remove_one(col))
                                    determinant_list[R, I](det_row_replace_entry[R, I](entry, r, det_row_fn_add[R, I](g1, g2)), tail, cols2.remove_one(col)) =
                                        determinant_list[R, I](det_row_replace_entry[R, I](entry, r, g1), tail, cols2.remove_one(col))
                                    det_row_sum_term[R, I](det_row_replace_entry[R, I](entry, r, det_row_fn_add[R, I](g1, g2)), head, tail, cols2, col) =
                                        (g1(col) + g2(col)) * alternating_sign[R](list_index_or_zero[I](cols2, col)) *
                                            determinant_list[R, I](det_row_replace_entry[R, I](entry, r, g1), tail, cols2.remove_one(col))
                                    (g1(col) + g2(col)) * alternating_sign[R](list_index_or_zero[I](cols2, col)) *
                                        determinant_list[R, I](det_row_replace_entry[R, I](entry, r, g1), tail, cols2.remove_one(col)) =
                                        g1(col) * alternating_sign[R](list_index_or_zero[I](cols2, col)) *
                                            determinant_list[R, I](det_row_replace_entry[R, I](entry, r, g1), tail, cols2.remove_one(col)) +
                                        g2(col) * alternating_sign[R](list_index_or_zero[I](cols2, col)) *
                                            determinant_list[R, I](det_row_replace_entry[R, I](entry, r, g1), tail, cols2.remove_one(col))
                                    det_row_sum_term[R, I](det_row_replace_entry[R, I](entry, r, g1), head, tail, cols2, col) =
                                        det_row_replace_entry[R, I](entry, r, g1, head, col) *
                                            alternating_sign[R](list_index_or_zero[I](cols2, col)) *
                                            determinant_list[R, I](det_row_replace_entry[R, I](entry, r, g1), tail, cols2.remove_one(col))
                                    det_row_replace_entry[R, I](entry, r, g1, head, col) = g1(col)
                                    det_row_sum_term[R, I](det_row_replace_entry[R, I](entry, r, g1), head, tail, cols2, col) =
                                        g1(col) * alternating_sign[R](list_index_or_zero[I](cols2, col)) *
                                            determinant_list[R, I](det_row_replace_entry[R, I](entry, r, g1), tail, cols2.remove_one(col))
                                    det_row_sum_term[R, I](det_row_replace_entry[R, I](entry, r, g2), head, tail, cols2, col) =
                                        det_row_replace_entry[R, I](entry, r, g2, head, col) *
                                            alternating_sign[R](list_index_or_zero[I](cols2, col)) *
                                            determinant_list[R, I](det_row_replace_entry[R, I](entry, r, g2), tail, cols2.remove_one(col))
                                    det_row_replace_entry[R, I](entry, r, g2, head, col) = g2(col)
                                    determinant_list_row_absent[R, I](entry, g2, r, tail, cols2.remove_one(col))
                                    determinant_list[R, I](det_row_replace_entry[R, I](entry, r, g2), tail, cols2.remove_one(col)) =
                                        determinant_list[R, I](entry, tail, cols2.remove_one(col))
                                    determinant_list_row_absent[R, I](entry, g1, r, tail, cols2.remove_one(col))
                                    determinant_list[R, I](det_row_replace_entry[R, I](entry, r, g1), tail, cols2.remove_one(col)) =
                                        determinant_list[R, I](entry, tail, cols2.remove_one(col))
                                    determinant_list[R, I](det_row_replace_entry[R, I](entry, r, g2), tail, cols2.remove_one(col)) =
                                        determinant_list[R, I](det_row_replace_entry[R, I](entry, r, g1), tail, cols2.remove_one(col))
                                    g2(col) * alternating_sign[R](list_index_or_zero[I](cols2, col)) *
                                        determinant_list[R, I](det_row_replace_entry[R, I](entry, r, g2), tail, cols2.remove_one(col)) =
                                        g2(col) * alternating_sign[R](list_index_or_zero[I](cols2, col)) *
                                            determinant_list[R, I](det_row_replace_entry[R, I](entry, r, g1), tail, cols2.remove_one(col))
                                    det_row_sum_term[R, I](det_row_replace_entry[R, I](entry, r, g2), head, tail, cols2, col) =
                                        g2(col) * alternating_sign[R](list_index_or_zero[I](cols2, col)) *
                                            determinant_list[R, I](det_row_replace_entry[R, I](entry, r, g1), tail, cols2.remove_one(col))
                                    det_row_sum_term[R, I](det_row_replace_entry[R, I](entry, r, det_row_fn_add[R, I](g1, g2)), head, tail, cols2, col) =
                                        det_row_sum_term[R, I](det_row_replace_entry[R, I](entry, r, g1), head, tail, cols2, col) +
                                            det_row_sum_term[R, I](det_row_replace_entry[R, I](entry, r, g2), head, tail, cols2, col)
                                }
                            }
                            sum_map_of_pointwise[I, R](cols2,
                                det_row_sum_term[R, I](det_row_replace_entry[R, I](entry, r, det_row_fn_add[R, I](g1, g2)), head, tail, cols2),
                                function(col: I) {
                                    det_row_sum_term[R, I](det_row_replace_entry[R, I](entry, r, g1), head, tail, cols2, col) +
                                        det_row_sum_term[R, I](det_row_replace_entry[R, I](entry, r, g2), head, tail, cols2, col)
                                })
                            sum[R](map[I, R](cols2, det_row_sum_term[R, I](det_row_replace_entry[R, I](entry, r, det_row_fn_add[R, I](g1, g2)), head, tail, cols2))) =
                                sum[R](map[I, R](cols2, function(col: I) {
                                    det_row_sum_term[R, I](det_row_replace_entry[R, I](entry, r, g1), head, tail, cols2, col) +
                                        det_row_sum_term[R, I](det_row_replace_entry[R, I](entry, r, g2), head, tail, cols2, col)
                                }))
                            sum_map_add_fn[I, R](cols2,
                                det_row_sum_term[R, I](det_row_replace_entry[R, I](entry, r, g1), head, tail, cols2),
                                det_row_sum_term[R, I](det_row_replace_entry[R, I](entry, r, g2), head, tail, cols2))
                            sum[R](map[I, R](cols2, function(col: I) {
                                det_row_sum_term[R, I](det_row_replace_entry[R, I](entry, r, g1), head, tail, cols2, col) +
                                    det_row_sum_term[R, I](det_row_replace_entry[R, I](entry, r, g2), head, tail, cols2, col)
                            })) =
                                sum[R](map[I, R](cols2, det_row_sum_term[R, I](det_row_replace_entry[R, I](entry, r, g1), head, tail, cols2))) +
                                    sum[R](map[I, R](cols2, det_row_sum_term[R, I](det_row_replace_entry[R, I](entry, r, g2), head, tail, cols2)))
                            sum[R](map[I, R](cols2, det_row_sum_term[R, I](det_row_replace_entry[R, I](entry, r, det_row_fn_add[R, I](g1, g2)), head, tail, cols2))) =
                                sum[R](map[I, R](cols2, det_row_sum_term[R, I](det_row_replace_entry[R, I](entry, r, g1), head, tail, cols2))) +
                                    sum[R](map[I, R](cols2, det_row_sum_term[R, I](det_row_replace_entry[R, I](entry, r, g2), head, tail, cols2)))
                            determinant_list[R, I](det_row_replace_entry[R, I](entry, r, det_row_fn_add[R, I](g1, g2)), List.cons(head, tail), cols2) =
                                determinant_list[R, I](det_row_replace_entry[R, I](entry, r, g1), List.cons(head, tail), cols2) +
                                    determinant_list[R, I](det_row_replace_entry[R, I](entry, r, g2), List.cons(head, tail), cols2)
                        }
                    }
                    if head != r {
                        remove_one_cons_neq(head, tail, r)
                        List.cons(head, tail).remove_one(r) = List.cons(head, tail.remove_one(r))
                        tail.contains(r)
                        not List.cons(head, tail.remove_one(r)).contains(r)
                        not tail.remove_one(r).contains(r)
                        p(tail) = (tail.contains(r) and not tail.remove_one(r).contains(r) implies
                            forall(cols3: List[I]) {
                                determinant_list[R, I](det_row_replace_entry[R, I](entry, r, det_row_fn_add[R, I](g1, g2)), tail, cols3) =
                                    determinant_list[R, I](det_row_replace_entry[R, I](entry, r, g1), tail, cols3) +
                                        determinant_list[R, I](det_row_replace_entry[R, I](entry, r, g2), tail, cols3)
                            })
                        forall(cols3: List[I]) {
                            determinant_list[R, I](det_row_replace_entry[R, I](entry, r, det_row_fn_add[R, I](g1, g2)), tail, cols3) =
                                determinant_list[R, I](det_row_replace_entry[R, I](entry, r, g1), tail, cols3) +
                                    determinant_list[R, I](det_row_replace_entry[R, I](entry, r, g2), tail, cols3)
                        }
                        forall(cols2: List[I]) {
                            determinant_list_cons_rows_term[R, I](det_row_replace_entry[R, I](entry, r, det_row_fn_add[R, I](g1, g2)), head, tail, cols2)
                            determinant_list[R, I](det_row_replace_entry[R, I](entry, r, det_row_fn_add[R, I](g1, g2)), List.cons(head, tail), cols2) =
                                sum[R](map[I, R](cols2, det_row_sum_term[R, I](det_row_replace_entry[R, I](entry, r, det_row_fn_add[R, I](g1, g2)), head, tail, cols2)))
                            determinant_list_cons_rows_term[R, I](det_row_replace_entry[R, I](entry, r, g1), head, tail, cols2)
                            determinant_list[R, I](det_row_replace_entry[R, I](entry, r, g1), List.cons(head, tail), cols2) =
                                sum[R](map[I, R](cols2, det_row_sum_term[R, I](det_row_replace_entry[R, I](entry, r, g1), head, tail, cols2)))
                            determinant_list_cons_rows_term[R, I](det_row_replace_entry[R, I](entry, r, g2), head, tail, cols2)
                            determinant_list[R, I](det_row_replace_entry[R, I](entry, r, g2), List.cons(head, tail), cols2) =
                                sum[R](map[I, R](cols2, det_row_sum_term[R, I](det_row_replace_entry[R, I](entry, r, g2), head, tail, cols2)))
                            forall(col: I) {
                                if cols2.contains(col) {
                                    det_row_sum_term[R, I](det_row_replace_entry[R, I](entry, r, det_row_fn_add[R, I](g1, g2)), head, tail, cols2, col) =
                                        det_row_replace_entry[R, I](entry, r, det_row_fn_add[R, I](g1, g2), head, col) *
                                            alternating_sign[R](list_index_or_zero[I](cols2, col)) *
                                            determinant_list[R, I](det_row_replace_entry[R, I](entry, r, det_row_fn_add[R, I](g1, g2)), tail, cols2.remove_one(col))
                                    det_row_replace_entry[R, I](entry, r, det_row_fn_add[R, I](g1, g2), head, col) = entry(head, col)
                                    determinant_list[R, I](det_row_replace_entry[R, I](entry, r, det_row_fn_add[R, I](g1, g2)), tail, cols2.remove_one(col)) =
                                        determinant_list[R, I](det_row_replace_entry[R, I](entry, r, g1), tail, cols2.remove_one(col)) +
                                            determinant_list[R, I](det_row_replace_entry[R, I](entry, r, g2), tail, cols2.remove_one(col))
                                    det_row_sum_term[R, I](det_row_replace_entry[R, I](entry, r, det_row_fn_add[R, I](g1, g2)), head, tail, cols2, col) =
                                        entry(head, col) * alternating_sign[R](list_index_or_zero[I](cols2, col)) *
                                            (determinant_list[R, I](det_row_replace_entry[R, I](entry, r, g1), tail, cols2.remove_one(col)) +
                                                determinant_list[R, I](det_row_replace_entry[R, I](entry, r, g2), tail, cols2.remove_one(col)))
                                    entry(head, col) * alternating_sign[R](list_index_or_zero[I](cols2, col)) *
                                        (determinant_list[R, I](det_row_replace_entry[R, I](entry, r, g1), tail, cols2.remove_one(col)) +
                                            determinant_list[R, I](det_row_replace_entry[R, I](entry, r, g2), tail, cols2.remove_one(col))) =
                                        entry(head, col) * alternating_sign[R](list_index_or_zero[I](cols2, col)) *
                                            determinant_list[R, I](det_row_replace_entry[R, I](entry, r, g1), tail, cols2.remove_one(col)) +
                                        entry(head, col) * alternating_sign[R](list_index_or_zero[I](cols2, col)) *
                                            determinant_list[R, I](det_row_replace_entry[R, I](entry, r, g2), tail, cols2.remove_one(col))
                                    det_row_sum_term[R, I](det_row_replace_entry[R, I](entry, r, g1), head, tail, cols2, col) =
                                        det_row_replace_entry[R, I](entry, r, g1, head, col) *
                                            alternating_sign[R](list_index_or_zero[I](cols2, col)) *
                                            determinant_list[R, I](det_row_replace_entry[R, I](entry, r, g1), tail, cols2.remove_one(col))
                                    det_row_replace_entry[R, I](entry, r, g1, head, col) = entry(head, col)
                                    det_row_sum_term[R, I](det_row_replace_entry[R, I](entry, r, g1), head, tail, cols2, col) =
                                        entry(head, col) * alternating_sign[R](list_index_or_zero[I](cols2, col)) *
                                            determinant_list[R, I](det_row_replace_entry[R, I](entry, r, g1), tail, cols2.remove_one(col))
                                    det_row_sum_term[R, I](det_row_replace_entry[R, I](entry, r, g2), head, tail, cols2, col) =
                                        det_row_replace_entry[R, I](entry, r, g2, head, col) *
                                            alternating_sign[R](list_index_or_zero[I](cols2, col)) *
                                            determinant_list[R, I](det_row_replace_entry[R, I](entry, r, g2), tail, cols2.remove_one(col))
                                    det_row_replace_entry[R, I](entry, r, g2, head, col) = entry(head, col)
                                    det_row_sum_term[R, I](det_row_replace_entry[R, I](entry, r, g2), head, tail, cols2, col) =
                                        entry(head, col) * alternating_sign[R](list_index_or_zero[I](cols2, col)) *
                                            determinant_list[R, I](det_row_replace_entry[R, I](entry, r, g2), tail, cols2.remove_one(col))
                                    det_row_sum_term[R, I](det_row_replace_entry[R, I](entry, r, det_row_fn_add[R, I](g1, g2)), head, tail, cols2, col) =
                                        det_row_sum_term[R, I](det_row_replace_entry[R, I](entry, r, g1), head, tail, cols2, col) +
                                            det_row_sum_term[R, I](det_row_replace_entry[R, I](entry, r, g2), head, tail, cols2, col)
                                }
                            }
                            sum_map_of_pointwise[I, R](cols2,
                                det_row_sum_term[R, I](det_row_replace_entry[R, I](entry, r, det_row_fn_add[R, I](g1, g2)), head, tail, cols2),
                                function(col: I) {
                                    det_row_sum_term[R, I](det_row_replace_entry[R, I](entry, r, g1), head, tail, cols2, col) +
                                        det_row_sum_term[R, I](det_row_replace_entry[R, I](entry, r, g2), head, tail, cols2, col)
                                })
                            sum[R](map[I, R](cols2, det_row_sum_term[R, I](det_row_replace_entry[R, I](entry, r, det_row_fn_add[R, I](g1, g2)), head, tail, cols2))) =
                                sum[R](map[I, R](cols2, function(col: I) {
                                    det_row_sum_term[R, I](det_row_replace_entry[R, I](entry, r, g1), head, tail, cols2, col) +
                                        det_row_sum_term[R, I](det_row_replace_entry[R, I](entry, r, g2), head, tail, cols2, col)
                                }))
                            sum_map_add_fn[I, R](cols2,
                                det_row_sum_term[R, I](det_row_replace_entry[R, I](entry, r, g1), head, tail, cols2),
                                det_row_sum_term[R, I](det_row_replace_entry[R, I](entry, r, g2), head, tail, cols2))
                            sum[R](map[I, R](cols2, function(col: I) {
                                det_row_sum_term[R, I](det_row_replace_entry[R, I](entry, r, g1), head, tail, cols2, col) +
                                    det_row_sum_term[R, I](det_row_replace_entry[R, I](entry, r, g2), head, tail, cols2, col)
                            })) =
                                sum[R](map[I, R](cols2, det_row_sum_term[R, I](det_row_replace_entry[R, I](entry, r, g1), head, tail, cols2))) +
                                    sum[R](map[I, R](cols2, det_row_sum_term[R, I](det_row_replace_entry[R, I](entry, r, g2), head, tail, cols2)))
                            sum[R](map[I, R](cols2, det_row_sum_term[R, I](det_row_replace_entry[R, I](entry, r, det_row_fn_add[R, I](g1, g2)), head, tail, cols2))) =
                                sum[R](map[I, R](cols2, det_row_sum_term[R, I](det_row_replace_entry[R, I](entry, r, g1), head, tail, cols2))) +
                                    sum[R](map[I, R](cols2, det_row_sum_term[R, I](det_row_replace_entry[R, I](entry, r, g2), head, tail, cols2)))
                            determinant_list[R, I](det_row_replace_entry[R, I](entry, r, det_row_fn_add[R, I](g1, g2)), List.cons(head, tail), cols2) =
                                determinant_list[R, I](det_row_replace_entry[R, I](entry, r, g1), List.cons(head, tail), cols2) +
                                    determinant_list[R, I](det_row_replace_entry[R, I](entry, r, g2), List.cons(head, tail), cols2)
                        }
                    }
                    forall(cols2: List[I]) {
                        determinant_list[R, I](det_row_replace_entry[R, I](entry, r, det_row_fn_add[R, I](g1, g2)), List.cons(head, tail), cols2) =
                            determinant_list[R, I](det_row_replace_entry[R, I](entry, r, g1), List.cons(head, tail), cols2) +
                                determinant_list[R, I](det_row_replace_entry[R, I](entry, r, g2), List.cons(head, tail), cols2)
                    }
                }
                p(tail) implies p(List.cons(head, tail))
            }
        }
        p(List.nil[I]) and forall(head: I, tail: List[I]) { p(tail) implies p(List.cons(head, tail)) }
        List.induction(p)
        forall(l: List[I]) { p(l) }
        p(rows)
        forall(cols2: List[I]) {
            determinant_list[R, I](det_row_replace_entry[R, I](entry, r, det_row_fn_add[R, I](g1, g2)), rows, cols2) =
                determinant_list[R, I](det_row_replace_entry[R, I](entry, r, g1), rows, cols2) +
                    determinant_list[R, I](det_row_replace_entry[R, I](entry, r, g2), rows, cols2)
        }
        determinant_list[R, I](det_row_replace_entry[R, I](entry, r, det_row_fn_add[R, I](g1, g2)), rows, cols) =
            determinant_list[R, I](det_row_replace_entry[R, I](entry, r, g1), rows, cols) +
                determinant_list[R, I](det_row_replace_entry[R, I](entry, r, g2), rows, cols)
    }
}

/// A determinant-list is homogeneous in a single row: replacing the row `r` of
/// an entry function by a scalar multiple of a row function multiplies the
/// determinant by the scalar, provided the row `r` occurs exactly once in the
/// row list.
theorem determinant_list_row_smul[R: CommRing, I](entry: (I, I) -> R, c: R, g: (I) -> R, r: I, rows: List[I], cols: List[I]) {
    rows.contains(r) and not rows.remove_one(r).contains(r) implies
        determinant_list[R, I](det_row_replace_entry[R, I](entry, r, det_row_fn_smul[R, I](c, g)), rows, cols) =
            c * determinant_list[R, I](det_row_replace_entry[R, I](entry, r, g), rows, cols)
} by {
    if rows.contains(r) and not rows.remove_one(r).contains(r) {
        define p(l: List[I]) -> Bool {
            l.contains(r) and not l.remove_one(r).contains(r) implies
                forall(cols2: List[I]) {
                    determinant_list[R, I](det_row_replace_entry[R, I](entry, r, det_row_fn_smul[R, I](c, g)), l, cols2) =
                        c * determinant_list[R, I](det_row_replace_entry[R, I](entry, r, g), l, cols2)
                }
        }
        if List.nil[I].contains(r) and not List.nil[I].remove_one(r).contains(r) {
            false
        }
        p(List.nil[I])
        forall(head: I, tail: List[I]) {
            if p(tail) {
                if List.cons(head, tail).contains(r) and not List.cons(head, tail).remove_one(r).contains(r) {
                    if head = r {
                        remove_one_cons_eq(head, tail)
                        List.cons(head, tail).remove_one(r) = tail
                        not tail.contains(r)
                        forall(cols2: List[I]) {
                            determinant_list_row_absent[R, I](entry, det_row_fn_smul[R, I](c, g), r, tail, cols2)
                            determinant_list[R, I](det_row_replace_entry[R, I](entry, r, det_row_fn_smul[R, I](c, g)), tail, cols2) =
                                determinant_list[R, I](entry, tail, cols2)
                            determinant_list_row_absent[R, I](entry, g, r, tail, cols2)
                            determinant_list[R, I](det_row_replace_entry[R, I](entry, r, g), tail, cols2) =
                                determinant_list[R, I](entry, tail, cols2)
                            determinant_list[R, I](det_row_replace_entry[R, I](entry, r, det_row_fn_smul[R, I](c, g)), tail, cols2) =
                                determinant_list[R, I](det_row_replace_entry[R, I](entry, r, g), tail, cols2)
                            determinant_list_cons_rows_term[R, I](det_row_replace_entry[R, I](entry, r, det_row_fn_smul[R, I](c, g)), head, tail, cols2)
                            determinant_list[R, I](det_row_replace_entry[R, I](entry, r, det_row_fn_smul[R, I](c, g)), List.cons(head, tail), cols2) =
                                sum[R](map[I, R](cols2, det_row_sum_term[R, I](det_row_replace_entry[R, I](entry, r, det_row_fn_smul[R, I](c, g)), head, tail, cols2)))
                            determinant_list_cons_rows_term[R, I](det_row_replace_entry[R, I](entry, r, g), head, tail, cols2)
                            determinant_list[R, I](det_row_replace_entry[R, I](entry, r, g), List.cons(head, tail), cols2) =
                                sum[R](map[I, R](cols2, det_row_sum_term[R, I](det_row_replace_entry[R, I](entry, r, g), head, tail, cols2)))
                            forall(col: I) {
                                if cols2.contains(col) {
                                    det_row_sum_term[R, I](det_row_replace_entry[R, I](entry, r, det_row_fn_smul[R, I](c, g)), head, tail, cols2, col) =
                                        det_row_replace_entry[R, I](entry, r, det_row_fn_smul[R, I](c, g), head, col) *
                                            alternating_sign[R](list_index_or_zero[I](cols2, col)) *
                                            determinant_list[R, I](det_row_replace_entry[R, I](entry, r, det_row_fn_smul[R, I](c, g)), tail, cols2.remove_one(col))
                                    det_row_replace_entry[R, I](entry, r, det_row_fn_smul[R, I](c, g), head, col) = det_row_fn_smul[R, I](c, g, col)
                                    det_row_fn_smul[R, I](c, g, col) = c * g(col)
                                    det_row_replace_entry[R, I](entry, r, det_row_fn_smul[R, I](c, g), head, col) = c * g(col)
                                    determinant_list_row_absent[R, I](entry, g, r, tail, cols2.remove_one(col))
                                    determinant_list[R, I](det_row_replace_entry[R, I](entry, r, g), tail, cols2.remove_one(col)) =
                                        determinant_list[R, I](entry, tail, cols2.remove_one(col))
                                    determinant_list_row_absent[R, I](entry, det_row_fn_smul[R, I](c, g), r, tail, cols2.remove_one(col))
                                    determinant_list[R, I](det_row_replace_entry[R, I](entry, r, det_row_fn_smul[R, I](c, g)), tail, cols2.remove_one(col)) =
                                        determinant_list[R, I](entry, tail, cols2.remove_one(col))
                                    det_row_sum_term[R, I](det_row_replace_entry[R, I](entry, r, det_row_fn_smul[R, I](c, g)), head, tail, cols2, col) =
                                        (c * g(col)) * alternating_sign[R](list_index_or_zero[I](cols2, col)) *
                                            determinant_list[R, I](det_row_replace_entry[R, I](entry, r, g), tail, cols2.remove_one(col))
                                    (c * g(col)) * alternating_sign[R](list_index_or_zero[I](cols2, col)) *
                                        determinant_list[R, I](det_row_replace_entry[R, I](entry, r, g), tail, cols2.remove_one(col)) =
                                        c * (g(col) * alternating_sign[R](list_index_or_zero[I](cols2, col)) *
                                            determinant_list[R, I](det_row_replace_entry[R, I](entry, r, g), tail, cols2.remove_one(col)))
                                    det_row_sum_term[R, I](det_row_replace_entry[R, I](entry, r, g), head, tail, cols2, col) =
                                        det_row_replace_entry[R, I](entry, r, g, head, col) *
                                            alternating_sign[R](list_index_or_zero[I](cols2, col)) *
                                            determinant_list[R, I](det_row_replace_entry[R, I](entry, r, g), tail, cols2.remove_one(col))
                                    det_row_replace_entry[R, I](entry, r, g, head, col) = g(col)
                                    det_row_sum_term[R, I](det_row_replace_entry[R, I](entry, r, g), head, tail, cols2, col) =
                                        g(col) * alternating_sign[R](list_index_or_zero[I](cols2, col)) *
                                            determinant_list[R, I](det_row_replace_entry[R, I](entry, r, g), tail, cols2.remove_one(col))
                                    det_row_sum_term[R, I](det_row_replace_entry[R, I](entry, r, det_row_fn_smul[R, I](c, g)), head, tail, cols2, col) =
                                        c * det_row_sum_term[R, I](det_row_replace_entry[R, I](entry, r, g), head, tail, cols2, col)
                                }
                            }
                            sum_map_of_pointwise[I, R](cols2,
                                det_row_sum_term[R, I](det_row_replace_entry[R, I](entry, r, det_row_fn_smul[R, I](c, g)), head, tail, cols2),
                                function(col: I) {
                                    c * det_row_sum_term[R, I](det_row_replace_entry[R, I](entry, r, g), head, tail, cols2, col)
                                })
                            sum[R](map[I, R](cols2, det_row_sum_term[R, I](det_row_replace_entry[R, I](entry, r, det_row_fn_smul[R, I](c, g)), head, tail, cols2))) =
                                sum[R](map[I, R](cols2, function(col: I) {
                                    c * det_row_sum_term[R, I](det_row_replace_entry[R, I](entry, r, g), head, tail, cols2, col)
                                }))
                            sum_map_scalar_mul[I, R](cols2, c, det_row_sum_term[R, I](det_row_replace_entry[R, I](entry, r, g), head, tail, cols2))
                            c * sum[R](map[I, R](cols2, det_row_sum_term[R, I](det_row_replace_entry[R, I](entry, r, g), head, tail, cols2))) =
                                sum[R](map[I, R](cols2, function(col: I) {
                                    c * det_row_sum_term[R, I](det_row_replace_entry[R, I](entry, r, g), head, tail, cols2, col)
                                }))
                            sum[R](map[I, R](cols2, det_row_sum_term[R, I](det_row_replace_entry[R, I](entry, r, det_row_fn_smul[R, I](c, g)), head, tail, cols2))) =
                                c * sum[R](map[I, R](cols2, det_row_sum_term[R, I](det_row_replace_entry[R, I](entry, r, g), head, tail, cols2)))
                            determinant_list[R, I](det_row_replace_entry[R, I](entry, r, det_row_fn_smul[R, I](c, g)), List.cons(head, tail), cols2) =
                                c * determinant_list[R, I](det_row_replace_entry[R, I](entry, r, g), List.cons(head, tail), cols2)
                        }
                    }
                    if head != r {
                        remove_one_cons_neq(head, tail, r)
                        List.cons(head, tail).remove_one(r) = List.cons(head, tail.remove_one(r))
                        tail.contains(r)
                        not List.cons(head, tail.remove_one(r)).contains(r)
                        not tail.remove_one(r).contains(r)
                        p(tail) = (tail.contains(r) and not tail.remove_one(r).contains(r) implies
                            forall(cols3: List[I]) {
                                determinant_list[R, I](det_row_replace_entry[R, I](entry, r, det_row_fn_smul[R, I](c, g)), tail, cols3) =
                                    c * determinant_list[R, I](det_row_replace_entry[R, I](entry, r, g), tail, cols3)
                            })
                        forall(cols3: List[I]) {
                            determinant_list[R, I](det_row_replace_entry[R, I](entry, r, det_row_fn_smul[R, I](c, g)), tail, cols3) =
                                c * determinant_list[R, I](det_row_replace_entry[R, I](entry, r, g), tail, cols3)
                        }
                        forall(cols2: List[I]) {
                            determinant_list_cons_rows_term[R, I](det_row_replace_entry[R, I](entry, r, det_row_fn_smul[R, I](c, g)), head, tail, cols2)
                            determinant_list[R, I](det_row_replace_entry[R, I](entry, r, det_row_fn_smul[R, I](c, g)), List.cons(head, tail), cols2) =
                                sum[R](map[I, R](cols2, det_row_sum_term[R, I](det_row_replace_entry[R, I](entry, r, det_row_fn_smul[R, I](c, g)), head, tail, cols2)))
                            determinant_list_cons_rows_term[R, I](det_row_replace_entry[R, I](entry, r, g), head, tail, cols2)
                            determinant_list[R, I](det_row_replace_entry[R, I](entry, r, g), List.cons(head, tail), cols2) =
                                sum[R](map[I, R](cols2, det_row_sum_term[R, I](det_row_replace_entry[R, I](entry, r, g), head, tail, cols2)))
                            forall(col: I) {
                                if cols2.contains(col) {
                                    det_row_sum_term[R, I](det_row_replace_entry[R, I](entry, r, det_row_fn_smul[R, I](c, g)), head, tail, cols2, col) =
                                        det_row_replace_entry[R, I](entry, r, det_row_fn_smul[R, I](c, g), head, col) *
                                            alternating_sign[R](list_index_or_zero[I](cols2, col)) *
                                            determinant_list[R, I](det_row_replace_entry[R, I](entry, r, det_row_fn_smul[R, I](c, g)), tail, cols2.remove_one(col))
                                    det_row_replace_entry[R, I](entry, r, det_row_fn_smul[R, I](c, g), head, col) = entry(head, col)
                                    determinant_list[R, I](det_row_replace_entry[R, I](entry, r, det_row_fn_smul[R, I](c, g)), tail, cols2.remove_one(col)) =
                                        c * determinant_list[R, I](det_row_replace_entry[R, I](entry, r, g), tail, cols2.remove_one(col))
                                    det_row_sum_term[R, I](det_row_replace_entry[R, I](entry, r, det_row_fn_smul[R, I](c, g)), head, tail, cols2, col) =
                                        entry(head, col) * alternating_sign[R](list_index_or_zero[I](cols2, col)) *
                                            (c * determinant_list[R, I](det_row_replace_entry[R, I](entry, r, g), tail, cols2.remove_one(col)))
                                    entry(head, col) * alternating_sign[R](list_index_or_zero[I](cols2, col)) *
                                        (c * determinant_list[R, I](det_row_replace_entry[R, I](entry, r, g), tail, cols2.remove_one(col))) =
                                        c * (entry(head, col) * alternating_sign[R](list_index_or_zero[I](cols2, col)) *
                                            determinant_list[R, I](det_row_replace_entry[R, I](entry, r, g), tail, cols2.remove_one(col)))
                                    det_row_sum_term[R, I](det_row_replace_entry[R, I](entry, r, g), head, tail, cols2, col) =
                                        det_row_replace_entry[R, I](entry, r, g, head, col) *
                                            alternating_sign[R](list_index_or_zero[I](cols2, col)) *
                                            determinant_list[R, I](det_row_replace_entry[R, I](entry, r, g), tail, cols2.remove_one(col))
                                    det_row_replace_entry[R, I](entry, r, g, head, col) = entry(head, col)
                                    det_row_sum_term[R, I](det_row_replace_entry[R, I](entry, r, g), head, tail, cols2, col) =
                                        entry(head, col) * alternating_sign[R](list_index_or_zero[I](cols2, col)) *
                                            determinant_list[R, I](det_row_replace_entry[R, I](entry, r, g), tail, cols2.remove_one(col))
                                    det_row_sum_term[R, I](det_row_replace_entry[R, I](entry, r, det_row_fn_smul[R, I](c, g)), head, tail, cols2, col) =
                                        c * det_row_sum_term[R, I](det_row_replace_entry[R, I](entry, r, g), head, tail, cols2, col)
                                }
                            }
                            sum_map_of_pointwise[I, R](cols2,
                                det_row_sum_term[R, I](det_row_replace_entry[R, I](entry, r, det_row_fn_smul[R, I](c, g)), head, tail, cols2),
                                function(col: I) {
                                    c * det_row_sum_term[R, I](det_row_replace_entry[R, I](entry, r, g), head, tail, cols2, col)
                                })
                            sum[R](map[I, R](cols2, det_row_sum_term[R, I](det_row_replace_entry[R, I](entry, r, det_row_fn_smul[R, I](c, g)), head, tail, cols2))) =
                                sum[R](map[I, R](cols2, function(col: I) {
                                    c * det_row_sum_term[R, I](det_row_replace_entry[R, I](entry, r, g), head, tail, cols2, col)
                                }))
                            sum_map_scalar_mul[I, R](cols2, c, det_row_sum_term[R, I](det_row_replace_entry[R, I](entry, r, g), head, tail, cols2))
                            c * sum[R](map[I, R](cols2, det_row_sum_term[R, I](det_row_replace_entry[R, I](entry, r, g), head, tail, cols2))) =
                                sum[R](map[I, R](cols2, function(col: I) {
                                    c * det_row_sum_term[R, I](det_row_replace_entry[R, I](entry, r, g), head, tail, cols2, col)
                                }))
                            sum[R](map[I, R](cols2, det_row_sum_term[R, I](det_row_replace_entry[R, I](entry, r, det_row_fn_smul[R, I](c, g)), head, tail, cols2))) =
                                c * sum[R](map[I, R](cols2, det_row_sum_term[R, I](det_row_replace_entry[R, I](entry, r, g), head, tail, cols2)))
                            determinant_list[R, I](det_row_replace_entry[R, I](entry, r, det_row_fn_smul[R, I](c, g)), List.cons(head, tail), cols2) =
                                c * determinant_list[R, I](det_row_replace_entry[R, I](entry, r, g), List.cons(head, tail), cols2)
                        }
                    }
                    forall(cols2: List[I]) {
                        determinant_list[R, I](det_row_replace_entry[R, I](entry, r, det_row_fn_smul[R, I](c, g)), List.cons(head, tail), cols2) =
                            c * determinant_list[R, I](det_row_replace_entry[R, I](entry, r, g), List.cons(head, tail), cols2)
                    }
                }
                p(tail) implies p(List.cons(head, tail))
            }
        }
        p(List.nil[I]) and forall(head: I, tail: List[I]) { p(tail) implies p(List.cons(head, tail)) }
        List.induction(p)
        forall(l: List[I]) { p(l) }
        p(rows)
        forall(cols2: List[I]) {
            determinant_list[R, I](det_row_replace_entry[R, I](entry, r, det_row_fn_smul[R, I](c, g)), rows, cols2) =
                c * determinant_list[R, I](det_row_replace_entry[R, I](entry, r, g), rows, cols2)
        }
        determinant_list[R, I](det_row_replace_entry[R, I](entry, r, det_row_fn_smul[R, I](c, g)), rows, cols) =
            c * determinant_list[R, I](det_row_replace_entry[R, I](entry, r, g), rows, cols)
    }
}

/// The matrix obtained by replacing one row with a supplied row function.
define matrix_row_replace[R: Ring](n: Nat, a: Matrix[R, n.suc, n.suc], r: Fin[n.suc], g: (Fin[n.suc]) -> R) -> Matrix[R, n.suc, n.suc] {
    Matrix[R, n.suc, n.suc].new(det_row_replace_entry[R, Fin[n.suc]](square_matrix_entry_function[R](n.suc, a), r, g))
}

/// The canonical determinant is additive in a single row.
theorem matrix_det_suc_row_add[R: Ring](n: Nat, a: Matrix[R, n.suc, n.suc], r: Fin[n.suc], x: (Fin[n.suc]) -> R, y: (Fin[n.suc]) -> R) {
    matrix_det_suc[R](n, matrix_row_replace[R](n, a, r, det_row_fn_add[R, Fin[n.suc]](x, y))) =
        matrix_det_suc[R](n, matrix_row_replace[R](n, a, r, x)) +
            matrix_det_suc[R](n, matrix_row_replace[R](n, a, r, y))
} by {
    fin_enum_suc_contains(n, r)
    fin_enum_suc(n).contains(r)
    fin_enum_suc_is_unique(n)
    fin_enum_suc(n).is_unique
    remove_one_unique_not_contains_self[Fin[n.suc]](fin_enum_suc(n), r)
    not fin_enum_suc(n).remove_one(r).contains(r)
    determinant_list_row_add[R, Fin[n.suc]](square_matrix_entry_function[R](n.suc, a), x, y, r, fin_enum_suc(n), fin_enum_suc(n))
    determinant_list[R, Fin[n.suc]](det_row_replace_entry[R, Fin[n.suc]](square_matrix_entry_function[R](n.suc, a), r, det_row_fn_add[R, Fin[n.suc]](x, y)), fin_enum_suc(n), fin_enum_suc(n)) =
        determinant_list[R, Fin[n.suc]](det_row_replace_entry[R, Fin[n.suc]](square_matrix_entry_function[R](n.suc, a), r, x), fin_enum_suc(n), fin_enum_suc(n)) +
            determinant_list[R, Fin[n.suc]](det_row_replace_entry[R, Fin[n.suc]](square_matrix_entry_function[R](n.suc, a), r, y), fin_enum_suc(n), fin_enum_suc(n))
    forall(i: Fin[n.suc], c: Fin[n.suc]) {
        square_matrix_entry_function[R](n.suc, matrix_row_replace[R](n, a, r, det_row_fn_add[R, Fin[n.suc]](x, y)), i, c) =
            det_row_replace_entry[R, Fin[n.suc]](square_matrix_entry_function[R](n.suc, a), r, det_row_fn_add[R, Fin[n.suc]](x, y), i, c)
    }
    determinant_list_pointwise[R, Fin[n.suc]](
        square_matrix_entry_function[R](n.suc, matrix_row_replace[R](n, a, r, det_row_fn_add[R, Fin[n.suc]](x, y))),
        det_row_replace_entry[R, Fin[n.suc]](square_matrix_entry_function[R](n.suc, a), r, det_row_fn_add[R, Fin[n.suc]](x, y)),
        fin_enum_suc(n), fin_enum_suc(n))
    determinant_list[R, Fin[n.suc]](square_matrix_entry_function[R](n.suc, matrix_row_replace[R](n, a, r, det_row_fn_add[R, Fin[n.suc]](x, y))), fin_enum_suc(n), fin_enum_suc(n)) =
        determinant_list[R, Fin[n.suc]](det_row_replace_entry[R, Fin[n.suc]](square_matrix_entry_function[R](n.suc, a), r, det_row_fn_add[R, Fin[n.suc]](x, y)), fin_enum_suc(n), fin_enum_suc(n))
    forall(i: Fin[n.suc], c: Fin[n.suc]) {
        square_matrix_entry_function[R](n.suc, matrix_row_replace[R](n, a, r, x), i, c) =
            det_row_replace_entry[R, Fin[n.suc]](square_matrix_entry_function[R](n.suc, a), r, x, i, c)
    }
    determinant_list_pointwise[R, Fin[n.suc]](
        square_matrix_entry_function[R](n.suc, matrix_row_replace[R](n, a, r, x)),
        det_row_replace_entry[R, Fin[n.suc]](square_matrix_entry_function[R](n.suc, a), r, x),
        fin_enum_suc(n), fin_enum_suc(n))
    determinant_list[R, Fin[n.suc]](det_row_replace_entry[R, Fin[n.suc]](square_matrix_entry_function[R](n.suc, a), r, x), fin_enum_suc(n), fin_enum_suc(n)) =
        determinant_list[R, Fin[n.suc]](square_matrix_entry_function[R](n.suc, matrix_row_replace[R](n, a, r, x)), fin_enum_suc(n), fin_enum_suc(n))
    forall(i: Fin[n.suc], c: Fin[n.suc]) {
        square_matrix_entry_function[R](n.suc, matrix_row_replace[R](n, a, r, y), i, c) =
            det_row_replace_entry[R, Fin[n.suc]](square_matrix_entry_function[R](n.suc, a), r, y, i, c)
    }
    determinant_list_pointwise[R, Fin[n.suc]](
        square_matrix_entry_function[R](n.suc, matrix_row_replace[R](n, a, r, y)),
        det_row_replace_entry[R, Fin[n.suc]](square_matrix_entry_function[R](n.suc, a), r, y),
        fin_enum_suc(n), fin_enum_suc(n))
    determinant_list[R, Fin[n.suc]](det_row_replace_entry[R, Fin[n.suc]](square_matrix_entry_function[R](n.suc, a), r, y), fin_enum_suc(n), fin_enum_suc(n)) =
        determinant_list[R, Fin[n.suc]](square_matrix_entry_function[R](n.suc, matrix_row_replace[R](n, a, r, y)), fin_enum_suc(n), fin_enum_suc(n))
    determinant_list[R, Fin[n.suc]](det_row_replace_entry[R, Fin[n.suc]](square_matrix_entry_function[R](n.suc, a), r, det_row_fn_add[R, Fin[n.suc]](x, y)), fin_enum_suc(n), fin_enum_suc(n)) =
        determinant_list[R, Fin[n.suc]](square_matrix_entry_function[R](n.suc, matrix_row_replace[R](n, a, r, x)), fin_enum_suc(n), fin_enum_suc(n)) +
            determinant_list[R, Fin[n.suc]](square_matrix_entry_function[R](n.suc, matrix_row_replace[R](n, a, r, y)), fin_enum_suc(n), fin_enum_suc(n))
    matrix_det_suc_unfold[R](n, matrix_row_replace[R](n, a, r, det_row_fn_add[R, Fin[n.suc]](x, y)))
    matrix_det_suc[R](n, matrix_row_replace[R](n, a, r, det_row_fn_add[R, Fin[n.suc]](x, y))) =
        matrix_det_list[R](n.suc, matrix_row_replace[R](n, a, r, det_row_fn_add[R, Fin[n.suc]](x, y)), fin_enum_suc(n), fin_enum_suc(n))
    matrix_det_list[R](n.suc, matrix_row_replace[R](n, a, r, det_row_fn_add[R, Fin[n.suc]](x, y)), fin_enum_suc(n), fin_enum_suc(n)) =
        determinant_list[R, Fin[n.suc]](square_matrix_entry_function[R](n.suc, matrix_row_replace[R](n, a, r, det_row_fn_add[R, Fin[n.suc]](x, y))), fin_enum_suc(n), fin_enum_suc(n))
    matrix_det_suc[R](n, matrix_row_replace[R](n, a, r, det_row_fn_add[R, Fin[n.suc]](x, y))) =
        determinant_list[R, Fin[n.suc]](square_matrix_entry_function[R](n.suc, matrix_row_replace[R](n, a, r, det_row_fn_add[R, Fin[n.suc]](x, y))), fin_enum_suc(n), fin_enum_suc(n))
    matrix_det_suc_unfold[R](n, matrix_row_replace[R](n, a, r, x))
    matrix_det_suc[R](n, matrix_row_replace[R](n, a, r, x)) =
        matrix_det_list[R](n.suc, matrix_row_replace[R](n, a, r, x), fin_enum_suc(n), fin_enum_suc(n))
    matrix_det_list[R](n.suc, matrix_row_replace[R](n, a, r, x), fin_enum_suc(n), fin_enum_suc(n)) =
        determinant_list[R, Fin[n.suc]](square_matrix_entry_function[R](n.suc, matrix_row_replace[R](n, a, r, x)), fin_enum_suc(n), fin_enum_suc(n))
    matrix_det_suc[R](n, matrix_row_replace[R](n, a, r, x)) =
        determinant_list[R, Fin[n.suc]](square_matrix_entry_function[R](n.suc, matrix_row_replace[R](n, a, r, x)), fin_enum_suc(n), fin_enum_suc(n))
    matrix_det_suc_unfold[R](n, matrix_row_replace[R](n, a, r, y))
    matrix_det_suc[R](n, matrix_row_replace[R](n, a, r, y)) =
        matrix_det_list[R](n.suc, matrix_row_replace[R](n, a, r, y), fin_enum_suc(n), fin_enum_suc(n))
    matrix_det_list[R](n.suc, matrix_row_replace[R](n, a, r, y), fin_enum_suc(n), fin_enum_suc(n)) =
        determinant_list[R, Fin[n.suc]](square_matrix_entry_function[R](n.suc, matrix_row_replace[R](n, a, r, y)), fin_enum_suc(n), fin_enum_suc(n))
    matrix_det_suc[R](n, matrix_row_replace[R](n, a, r, y)) =
        determinant_list[R, Fin[n.suc]](square_matrix_entry_function[R](n.suc, matrix_row_replace[R](n, a, r, y)), fin_enum_suc(n), fin_enum_suc(n))
    matrix_det_suc[R](n, matrix_row_replace[R](n, a, r, det_row_fn_add[R, Fin[n.suc]](x, y))) =
        matrix_det_suc[R](n, matrix_row_replace[R](n, a, r, x)) +
            matrix_det_suc[R](n, matrix_row_replace[R](n, a, r, y))
}

/// The canonical determinant is homogeneous in a single row.
theorem matrix_det_suc_row_smul[R: CommRing](n: Nat, a: Matrix[R, n.suc, n.suc], r: Fin[n.suc], c: R, x: (Fin[n.suc]) -> R) {
    matrix_det_suc[R](n, matrix_row_replace[R](n, a, r, det_row_fn_smul[R, Fin[n.suc]](c, x))) =
        c * matrix_det_suc[R](n, matrix_row_replace[R](n, a, r, x))
} by {
    fin_enum_suc_contains(n, r)
    fin_enum_suc(n).contains(r)
    fin_enum_suc_is_unique(n)
    fin_enum_suc(n).is_unique
    remove_one_unique_not_contains_self[Fin[n.suc]](fin_enum_suc(n), r)
    not fin_enum_suc(n).remove_one(r).contains(r)
    determinant_list_row_smul[R, Fin[n.suc]](square_matrix_entry_function[R](n.suc, a), c, x, r, fin_enum_suc(n), fin_enum_suc(n))
    determinant_list[R, Fin[n.suc]](det_row_replace_entry[R, Fin[n.suc]](square_matrix_entry_function[R](n.suc, a), r, det_row_fn_smul[R, Fin[n.suc]](c, x)), fin_enum_suc(n), fin_enum_suc(n)) =
        c * determinant_list[R, Fin[n.suc]](det_row_replace_entry[R, Fin[n.suc]](square_matrix_entry_function[R](n.suc, a), r, x), fin_enum_suc(n), fin_enum_suc(n))
    forall(i: Fin[n.suc], c2: Fin[n.suc]) {
        square_matrix_entry_function[R](n.suc, matrix_row_replace[R](n, a, r, det_row_fn_smul[R, Fin[n.suc]](c, x)), i, c2) =
            det_row_replace_entry[R, Fin[n.suc]](square_matrix_entry_function[R](n.suc, a), r, det_row_fn_smul[R, Fin[n.suc]](c, x), i, c2)
    }
    determinant_list_pointwise[R, Fin[n.suc]](
        square_matrix_entry_function[R](n.suc, matrix_row_replace[R](n, a, r, det_row_fn_smul[R, Fin[n.suc]](c, x))),
        det_row_replace_entry[R, Fin[n.suc]](square_matrix_entry_function[R](n.suc, a), r, det_row_fn_smul[R, Fin[n.suc]](c, x)),
        fin_enum_suc(n), fin_enum_suc(n))
    determinant_list[R, Fin[n.suc]](square_matrix_entry_function[R](n.suc, matrix_row_replace[R](n, a, r, det_row_fn_smul[R, Fin[n.suc]](c, x))), fin_enum_suc(n), fin_enum_suc(n)) =
        determinant_list[R, Fin[n.suc]](det_row_replace_entry[R, Fin[n.suc]](square_matrix_entry_function[R](n.suc, a), r, det_row_fn_smul[R, Fin[n.suc]](c, x)), fin_enum_suc(n), fin_enum_suc(n))
    forall(i: Fin[n.suc], c2: Fin[n.suc]) {
        square_matrix_entry_function[R](n.suc, matrix_row_replace[R](n, a, r, x), i, c2) =
            det_row_replace_entry[R, Fin[n.suc]](square_matrix_entry_function[R](n.suc, a), r, x, i, c2)
    }
    determinant_list_pointwise[R, Fin[n.suc]](
        square_matrix_entry_function[R](n.suc, matrix_row_replace[R](n, a, r, x)),
        det_row_replace_entry[R, Fin[n.suc]](square_matrix_entry_function[R](n.suc, a), r, x),
        fin_enum_suc(n), fin_enum_suc(n))
    determinant_list[R, Fin[n.suc]](det_row_replace_entry[R, Fin[n.suc]](square_matrix_entry_function[R](n.suc, a), r, x), fin_enum_suc(n), fin_enum_suc(n)) =
        determinant_list[R, Fin[n.suc]](square_matrix_entry_function[R](n.suc, matrix_row_replace[R](n, a, r, x)), fin_enum_suc(n), fin_enum_suc(n))
    determinant_list[R, Fin[n.suc]](det_row_replace_entry[R, Fin[n.suc]](square_matrix_entry_function[R](n.suc, a), r, det_row_fn_smul[R, Fin[n.suc]](c, x)), fin_enum_suc(n), fin_enum_suc(n)) =
        c * determinant_list[R, Fin[n.suc]](square_matrix_entry_function[R](n.suc, matrix_row_replace[R](n, a, r, x)), fin_enum_suc(n), fin_enum_suc(n))
    matrix_det_suc_unfold[R](n, matrix_row_replace[R](n, a, r, det_row_fn_smul[R, Fin[n.suc]](c, x)))
    matrix_det_suc[R](n, matrix_row_replace[R](n, a, r, det_row_fn_smul[R, Fin[n.suc]](c, x))) =
        matrix_det_list[R](n.suc, matrix_row_replace[R](n, a, r, det_row_fn_smul[R, Fin[n.suc]](c, x)), fin_enum_suc(n), fin_enum_suc(n))
    matrix_det_list[R](n.suc, matrix_row_replace[R](n, a, r, det_row_fn_smul[R, Fin[n.suc]](c, x)), fin_enum_suc(n), fin_enum_suc(n)) =
        determinant_list[R, Fin[n.suc]](square_matrix_entry_function[R](n.suc, matrix_row_replace[R](n, a, r, det_row_fn_smul[R, Fin[n.suc]](c, x))), fin_enum_suc(n), fin_enum_suc(n))
    matrix_det_suc[R](n, matrix_row_replace[R](n, a, r, det_row_fn_smul[R, Fin[n.suc]](c, x))) =
        determinant_list[R, Fin[n.suc]](square_matrix_entry_function[R](n.suc, matrix_row_replace[R](n, a, r, det_row_fn_smul[R, Fin[n.suc]](c, x))), fin_enum_suc(n), fin_enum_suc(n))
    matrix_det_suc_unfold[R](n, matrix_row_replace[R](n, a, r, x))
    matrix_det_suc[R](n, matrix_row_replace[R](n, a, r, x)) =
        matrix_det_list[R](n.suc, matrix_row_replace[R](n, a, r, x), fin_enum_suc(n), fin_enum_suc(n))
    matrix_det_list[R](n.suc, matrix_row_replace[R](n, a, r, x), fin_enum_suc(n), fin_enum_suc(n)) =
        determinant_list[R, Fin[n.suc]](square_matrix_entry_function[R](n.suc, matrix_row_replace[R](n, a, r, x)), fin_enum_suc(n), fin_enum_suc(n))
    matrix_det_suc[R](n, matrix_row_replace[R](n, a, r, x)) =
        determinant_list[R, Fin[n.suc]](square_matrix_entry_function[R](n.suc, matrix_row_replace[R](n, a, r, x)), fin_enum_suc(n), fin_enum_suc(n))
    matrix_det_suc[R](n, matrix_row_replace[R](n, a, r, det_row_fn_smul[R, Fin[n.suc]](c, x))) =
        c * matrix_det_suc[R](n, matrix_row_replace[R](n, a, r, x))
}

// ---------------------------------------------------------------------------
// Status note (do not delete): multiplicative property of the determinant.
//
// The row-multilinearity lemmas above (determinant_list_row_add,
// determinant_list_row_smul, matrix_det_suc_row_add, matrix_det_suc_row_smul)
// are the foundation for proving det(A.B) = det(A).det(B) by the
// characterization route:
//
//   Let F_A(B) = matrix_det_suc(n, A.mul(B)).  If we know that any function
//   on (n+1)x(n+1) matrices that is (i) multilinear in every row, (ii)
//   alternating (equal rows give zero, swapping two rows negates), and (iii)
//   sends the identity to 1, is equal to matrix_det_suc, then F_A(B) =
//   matrix_det_suc(B).F_A(1) and multiplicativity follows.
//
// The missing pieces, in order of dependency:
//   1. The Leibniz formula for the list-based determinant:
//        determinant_list(e, rows, cols) = sum over permutations sigma of
//        sign(sigma) . product_i e(rows_i, cols_{sigma(i)}),
//      restricted to the canonical enumerations.  This requires the sign of a
//      permutation of Fin[n] (well-definedness, sign of a transposition is
//      -1, sign(sigma o tau) = sign(sigma).sign(tau)).  There is currently NO
//      permutation-sign infrastructure anywhere in the library (only
//      List.is_permutation, which is count-based multiset equality, and
//      alternating_sign for (-1)^k).  The existing row-swap lemmas cover only
//      the first two rows (determinant_list_swap_first_two_rows), and
//      generalising them needs exactly this theory.
//   2. General equal-rows: two equal rows give determinant zero.  The
//      natural proof pairs sigma with sigma o (ij) in the Leibniz sum; even
//      for equal rows in the first two positions the naive double-sum
//      cancellation only yields 2.det = 0, which is not zero in general
//      characteristic, so the unordered-pair formulation (or Leibniz) is
//      required.
//   3. The characterization theorem and the application to F_A.
//
// All three routes to det(A.B) = det(A).det(B) (characterization via
// multilinearity, direct Cauchy-Binet expansion, or Leibniz expansion of the
// product) pass through the Leibniz formula, so the sign-of-permutation
// theory is the true bottleneck, not the list-based definition of the
// determinant itself.
// ---------------------------------------------------------------------------
