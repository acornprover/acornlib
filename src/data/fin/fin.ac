from nat import Nat

/// Finite initial segment `{ 0, 1, ..., n - 1 }`.
structure Fin[n: Nat] {
    value: Nat
} constraint {
    value < n
}

/// Identity witness used to exercise top-level dependent `let ... satisfy`.
let fin_self(n: Nat, x: Fin[n]) -> result: Fin[n] satisfy {
    result = x
} by {
    x = x
}

attributes Fin[n: Nat] {
    /// Expose the ambient bound in a family-dependent attribute body.
    define bound(self) -> Nat {
        n
    }

    /// Rewrap through a top-level dependent satisfy function.
    define again(self) -> Fin[n] {
        fin_self(n, self)
    }
}

theorem value_lt_bound(n: Nat, x: Fin[n]) {
    x.value < n
}

theorem new_round_trip(n: Nat, x: Nat, y: Fin[n]) {
    Fin[n].new(x) = Option.some(y) implies y.value = x
}

theorem new_self(n: Nat, a: Fin[n]) {
    Fin[n].new(a.value) = Option.some(a)
}

/// Constructing a finite index from a natural number proves the natural is in range.
theorem fin_new_some_imp_lt(n: Nat, k: Nat, i: Fin[n]) {
    Fin[n].new(k) = Option.some(i) implies k < n
} by {
    if Fin[n].new(k) = Option.some(i) {
        new_round_trip(n, k, i)
        i.value < n
        k < n
    }
}

/// A natural number admits a `Fin[n]` constructor exactly when it is below `n`.
theorem fin_new_some_iff_lt(n: Nat, k: Nat) {
    (exists(i: Fin[n]) { Fin[n].new(k) = Option.some(i) }) = (k < n)
} by {
    if exists(i: Fin[n]) { Fin[n].new(k) = Option.some(i) } {
        let i: Fin[n] satisfy { Fin[n].new(k) = Option.some(i) }
        fin_new_some_imp_lt(n, k, i)
        k < n
    }
    if k < n {
        exists(i: Fin[n]) {
            Fin[n].new(k) = Option.some(i)
        }
    }
}

/// A strict natural bound gives a finite-index witness.
theorem fin_new_exists_of_lt(n: Nat, k: Nat) {
    k < n implies exists(i: Fin[n]) {
        Fin[n].new(k) = Option.some(i)
    }
} by {
    if k < n {
        exists(i: Fin[n]) {
            Fin[n].new(k) = Option.some(i)
        }
    }
}

theorem ext(n: Nat, a: Fin[n], b: Fin[n]) {
    a.value = b.value implies a = b
} by {
    if a.value = b.value {
        Option.some(a) = Option.some(b)
        some_injective[Fin[n]](a, b)
    }
}

theorem attribute_round_trip(n: Nat, x: Fin[n]) {
    x.bound = n and x.again = x
}

/// The dependent bound attribute is definitionally the ambient bound.
theorem fin_bound_eq(n: Nat, x: Fin[n]) {
    x.bound = n
} by {
}

/// Rewrapping a finite index through `again` is the identity.
theorem fin_again_eq(n: Nat, x: Fin[n]) {
    x.again = x
} by {
    attribute_round_trip(n, x)
}

/// The final element of `Fin[n + 1]`.
let fin_last(n: Nat) -> result: Fin[n.suc] satisfy {
    Fin[n.suc].new(n) = Option.some(result)
} by {
}

/// The first element of `Fin[n + 1]`.
let fin_zero(n: Nat) -> result: Fin[n.suc] satisfy {
    Fin[n.suc].new(Nat.0) = Option.some(result)
} by {
    Nat.0 < n.suc
}

/// The standard inclusion `Fin[n]` into `Fin[n + 1]`.
let fin_cast_suc(n: Nat, x: Fin[n]) -> result: Fin[n.suc] satisfy {
    Fin[n.suc].new(x.value) = Option.some(result)
} by {
    x.value < n.suc
}

/// The successor inclusion `Fin[n]` into `Fin[n + 1]`.
let fin_succ(n: Nat, x: Fin[n]) -> result: Fin[n.suc] satisfy {
    Fin[n.suc].new(x.value.suc) = Option.some(result)
} by {
    x.value < n
    x.value.suc <= n
    x.value.suc < n.suc
}

/// The natural index obtained by skipping one index in `Fin[n + 1]`.
define fin_skip_value(n: Nat, removed: Fin[n.suc], x: Fin[n]) -> Nat {
    if x.value < removed.value {
        x.value
    } else {
        x.value.suc
    }
}

/// The increasing inclusion `Fin[n] -> Fin[n + 1]` whose image omits `removed`.
define fin_skip(n: Nat, removed: Fin[n.suc], x: Fin[n]) -> Fin[n.suc] {
    if x.value < removed.value {
        fin_cast_suc(n, x)
    } else {
        fin_succ(n, x)
    }
}

attributes Fin[n: Nat] {
    /// True if this element is the final element of the finite initial segment.
    define is_last(self) -> Bool {
        self.value.suc = n
    }

    /// The standard inclusion into the next finite initial segment.
    define cast_suc(self) -> Fin[n.suc] {
        fin_cast_suc(n, self)
    }

    /// The increasing inclusion into `Fin[n + 1]` omitting `removed`.
    define skip(self, removed: Fin[n.suc]) -> Fin[n.suc] {
        fin_skip(n, removed, self)
    }
}

/// The last finite index has value `n`.
theorem fin_last_value(n: Nat) {
    fin_last(n).value = n
} by {
    new_round_trip(n.suc, n, fin_last(n))
}

/// The last finite index satisfies the `is_last` predicate.
theorem fin_last_is_last(n: Nat) {
    fin_last(n).is_last
} by {
    fin_last_value(n)
}

/// The `is_last` predicate is the natural successor-value equation.
theorem fin_is_last_iff_value_suc_eq(n: Nat, x: Fin[n]) {
    x.is_last = (x.value.suc = n)
} by {
}

/// The standard inclusion preserves the natural value.
theorem fin_cast_suc_value(n: Nat, x: Fin[n]) {
    fin_cast_suc(n, x).value = x.value
} by {
    new_round_trip(n.suc, x.value, fin_cast_suc(n, x))
}

/// The first finite index has value zero.
theorem fin_zero_value(n: Nat) {
    fin_zero(n).value = Nat.0
} by {
    new_round_trip(n.suc, Nat.0, fin_zero(n))
}

/// The first finite index is represented by natural index zero.
theorem fin_zero_new(n: Nat) {
    Fin[n.suc].new(Nat.0) = Option.some(fin_zero(n))
}

/// The final finite index is represented by natural index `n`.
theorem fin_last_new(n: Nat) {
    Fin[n.suc].new(n) = Option.some(fin_last(n))
}

/// The finite index represented by `k` in `Fin[n + 1]`, with zero as fallback outside the range.
define fin_of_nat_suc(n: Nat, k: Nat) -> Fin[n.suc] {
    match Fin[n.suc].new(k) {
        Option.none {
            fin_zero(n)
        }
        Option.some(i) {
            i
        }
    }
}

/// A bounded natural is represented by `fin_of_nat_suc`.
theorem fin_of_nat_suc_new_of_lt(n: Nat, k: Nat) {
    k < n.suc implies Fin[n.suc].new(k) = Option.some(fin_of_nat_suc(n, k))
} by {
    if k < n.suc {
        fin_new_exists_of_lt(n.suc, k)
        match Fin[n.suc].new(k) {
            Option.none {
                let i: Fin[n.suc] satisfy {
                    Fin[n.suc].new(k) = Option.some(i)
                }
            }
            Option.some(i) {
                fin_of_nat_suc(n, k) = i
                Fin[n.suc].new(k) = Option.some(fin_of_nat_suc(n, k))
            }
        }
    }
}

/// On bounded naturals, `fin_of_nat_suc` has the requested value.
theorem fin_of_nat_suc_value_of_lt(n: Nat, k: Nat) {
    k < n.suc implies fin_of_nat_suc(n, k).value = k
} by {
    if k < n.suc {
        fin_of_nat_suc_new_of_lt(n, k)
        new_round_trip(n.suc, k, fin_of_nat_suc(n, k))
    }
}

/// The successor inclusion increments the natural value.
theorem fin_succ_value(n: Nat, x: Fin[n]) {
    fin_succ(n, x).value = x.value.suc
} by {
    new_round_trip(n.suc, x.value.suc, fin_succ(n, x))
}

/// Skipping an index has the expected natural value.
theorem fin_skip_value_eq(n: Nat, removed: Fin[n.suc], x: Fin[n]) {
    fin_skip(n, removed, x).value = fin_skip_value(n, removed, x)
} by {
    if x.value < removed.value {
        fin_skip(n, removed, x) = fin_cast_suc(n, x)
        fin_cast_suc_value(n, x)
        fin_skip(n, removed, x).value = fin_skip_value(n, removed, x)
    } else {
        fin_skip(n, removed, x) = fin_succ(n, x)
        fin_succ_value(n, x)
        fin_skip(n, removed, x).value = fin_skip_value(n, removed, x)
    }
}

/// Skipping never lands on the omitted index.
theorem fin_skip_ne_removed(n: Nat, removed: Fin[n.suc], x: Fin[n]) {
    fin_skip(n, removed, x) != removed
} by {
    fin_skip_value_eq(n, removed, x)
    if fin_skip(n, removed, x) = removed {
        fin_skip_value(n, removed, x) = removed.value
        if x.value < removed.value {
            x.value = removed.value
            false
        } else {
            x.value.suc = removed.value
            false
        }
    }
}
