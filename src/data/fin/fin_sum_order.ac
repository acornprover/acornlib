from nat import Nat, lt_or_lte, lte_antisymm, lt_imp_lte_suc, lte_and_lt, lt_and_lte,
    lte_cancel_suc, only_zero_lte_zero
from order import lte_trans, lte_refl
from data.fin.fin import Fin
from ordered_field import OrderedField
from list import partial, partial_zero, partial_split_last
from data.fin.fin_sum import fin_sum, fin_sum_apply, fin_value_or_zero, fin_value_or_zero_value,
    fin_value_or_zero_of_not_lt

numerals Nat

/// A partial sum of nonnegative terms is nonnegative.
///
/// `src/fin_sum.ac` and the `partial` sums underneath it carry no order lemmas at all, so
/// this and the term bound below are the first ones. Both are what any argument comparing a
/// single entry against a row total needs.
///
/// The hypothesis is nonnegativity everywhere rather than below the bound. That is what the
/// zero-extended functions below satisfy, and it keeps the induction predicate free of the
/// index, without which the inductive step cannot discharge its own hypothesis.
theorem partial_nonneg[S: OrderedField](f: Nat -> S, n: Nat) {
    (forall(i: Nat) { S.0 <= f(i) }) implies S.0 <= partial(f, n)
} by {
    if forall(i: Nat) { S.0 <= f(i) } {
        define p(x: Nat) -> Bool {
            S.0 <= partial(f, x)
        }
        partial_zero(f)
        partial(f, Nat.0) = S.0
        S.0 <= S.0
        S.0 <= partial(f, Nat.0)
        p(Nat.0)
        forall(k: Nat) {
            if p(k) {
                S.0 <= partial(f, k)
                S.0 <= f(k)
                partial_split_last(f, k)
                partial(f, k.suc) = partial(f, k) + f(k)
                S.0 + S.0 = S.0
                S.0 + S.0 <= partial(f, k) + f(k)
                S.0 <= partial(f, k.suc)
                p(k.suc)
            }
            (p(k) implies p(k.suc))
        }
        p(Nat.0) and forall(k: Nat) {
            p(k) implies p(k.suc)
        }
        Nat.induction(p)
        p(n)
        S.0 <= partial(f, n)
    }
}

/// Partial sums of nonnegative terms grow with the bound.
theorem partial_mono[S: OrderedField](f: Nat -> S, m: Nat, n: Nat) {
    (forall(i: Nat) { S.0 <= f(i) }) and m <= n implies partial(f, m) <= partial(f, n)
} by {
    if forall(i: Nat) { S.0 <= f(i) } {
        define p(x: Nat) -> Bool {
            m <= x implies partial(f, m) <= partial(f, x)
        }
        if m <= Nat.0 {
            only_zero_lte_zero(m)
            m = Nat.0
            lte_refl(partial(f, Nat.0))
            partial(f, Nat.0) <= partial(f, Nat.0)
            partial(f, m) <= partial(f, Nat.0)
        }
        (m <= Nat.0 implies partial(f, m) <= partial(f, Nat.0))
        p(Nat.0)
        forall(k: Nat) {
            if p(k) {
                if m <= k.suc {
                    partial_split_last(f, k)
                    partial(f, k.suc) = partial(f, k) + f(k)
                    S.0 <= f(k)
                    partial(f, k) + S.0 <= partial(f, k) + f(k)
                    partial(f, k) + S.0 = partial(f, k)
                    partial(f, k) <= partial(f, k.suc)
                    if m <= k {
                        (m <= k implies partial(f, m) <= partial(f, k))
                        partial(f, m) <= partial(f, k)
                        lte_trans(partial(f, m), partial(f, k), partial(f, k.suc))
                        partial(f, m) <= partial(f, k.suc)
                    }
                    if not (m <= k) {
                        lt_or_lte(k, m)
                        k < m
                        lt_imp_lte_suc(k, m)
                        k.suc <= m
                        lte_antisymm(m, k.suc)
                        m = k.suc
                        lte_refl(partial(f, k.suc))
                        partial(f, k.suc) <= partial(f, k.suc)
                        partial(f, m) <= partial(f, k.suc)
                    }
                    partial(f, m) <= partial(f, k.suc)
                }
                (m <= k.suc implies partial(f, m) <= partial(f, k.suc))
                p(k.suc)
            }
            (p(k) implies p(k.suc))
        }
        p(Nat.0) and forall(k: Nat) {
            p(k) implies p(k.suc)
        }
        Nat.induction(p)
        p(n)
        (m <= n implies partial(f, m) <= partial(f, n))
    }
}

/// The last term of a partial sum of nonnegative terms is at most the sum.
///
/// Everything before it is nonnegative, so it can only add.
theorem partial_last_le[S: OrderedField](f: Nat -> S, j: Nat) {
    (forall(i: Nat) { S.0 <= f(i) }) implies f(j) <= partial(f, j.suc)
} by {
    if forall(i: Nat) { S.0 <= f(i) } {
        partial_split_last(f, j)
        partial(f, j.suc) = partial(f, j) + f(j)
        partial_nonneg[S](f, j)
        S.0 <= partial(f, j)
        S.0 + f(j) <= partial(f, j) + f(j)
        S.0 + f(j) = f(j)
        f(j) <= partial(f, j.suc)
    }
}

/// A single term of a partial sum of nonnegative terms is at most the sum.
///
/// The term is already at most the sum that stops just after it, and partial sums of
/// nonnegative terms only grow. Splitting the argument this way avoids an induction whose
/// base case would be a vacuously true implication, which the verifier rejects.
theorem partial_term_le[S: OrderedField](f: Nat -> S, n: Nat, j: Nat) {
    (forall(i: Nat) { S.0 <= f(i) }) and j < n implies f(j) <= partial(f, n)
} by {
    if (forall(i: Nat) { S.0 <= f(i) }) and j < n {
        partial_last_le[S](f, j)
        f(j) <= partial(f, j.suc)
        lt_imp_lte_suc(j, n)
        j.suc <= n
        partial_mono[S](f, j.suc, n)
        partial(f, j.suc) <= partial(f, n)
        lte_trans(f(j), partial(f, j.suc), partial(f, n))
        f(j) <= partial(f, n)
    }
}

/// The zero extension of a nonnegative function is nonnegative.
///
/// Outside the range the extension is zero, and inside it is one of the original values.
theorem fin_value_or_zero_nonneg[S: OrderedField](n: Nat, f: Fin[n] -> S, k: Nat) {
    (forall(i: Fin[n]) { S.0 <= f(i) }) implies S.0 <= fin_value_or_zero[S](n, f, k)
} by {
    if forall(i: Fin[n]) { S.0 <= f(i) } {
        if k < n {
            let (i: Fin[n]) satisfy {
                Fin[n].new(k) = Option.some(i)
            }
            fin_value_or_zero_value[S](n, f, i)
            i.value = k
            fin_value_or_zero[S](n, f, k) = f(i)
            S.0 <= f(i)
            S.0 <= fin_value_or_zero[S](n, f, k)
        }
        if not (k < n) {
            fin_value_or_zero_of_not_lt[S](n, f, k)
            fin_value_or_zero[S](n, f, k) = S.0
            S.0 <= fin_value_or_zero[S](n, f, k)
        }
        S.0 <= fin_value_or_zero[S](n, f, k)
    }
}

/// A finite sum of nonnegative terms is nonnegative.
theorem fin_sum_nonneg[S: OrderedField](n: Nat, f: Fin[n] -> S) {
    (forall(i: Fin[n]) { S.0 <= f(i) }) implies S.0 <= fin_sum[S](n, f)
} by {
    if forall(i: Fin[n]) { S.0 <= f(i) } {
        forall(k: Nat) {
            fin_value_or_zero_nonneg[S](n, f, k)
            S.0 <= fin_value_or_zero[S](n, f, k)
        }
        partial_nonneg[S](fin_value_or_zero[S](n, f), n)
        S.0 <= partial(fin_value_or_zero[S](n, f), n)
        fin_sum_apply[S](n, f)
        fin_sum[S](n, f) = partial(fin_value_or_zero[S](n, f), n)
        S.0 <= fin_sum[S](n, f)
    }
}

/// A single term of a finite sum of nonnegative terms is at most the sum.
///
/// This is the bound that lets a matrix entry be compared against its row total, and so the
/// reason the order lemmas above were added.
theorem fin_sum_term_le[S: OrderedField](n: Nat, f: Fin[n] -> S, target: Fin[n]) {
    (forall(i: Fin[n]) { S.0 <= f(i) }) implies f(target) <= fin_sum[S](n, f)
} by {
    if forall(i: Fin[n]) { S.0 <= f(i) } {
        forall(k: Nat) {
            fin_value_or_zero_nonneg[S](n, f, k)
            S.0 <= fin_value_or_zero[S](n, f, k)
        }
        target.value < n
        partial_term_le[S](fin_value_or_zero[S](n, f), n, target.value)
        (fin_value_or_zero[S](n, f, target.value)
            <= partial(fin_value_or_zero[S](n, f), n))
        fin_value_or_zero_value[S](n, f, target)
        fin_value_or_zero[S](n, f, target.value) = f(target)
        fin_sum_apply[S](n, f)
        fin_sum[S](n, f) = partial(fin_value_or_zero[S](n, f), n)
        f(target) <= fin_sum[S](n, f)
    }
}

/// Partial sums are monotone in the summand.
///
/// Comparing termwise compares the totals. Stated with the comparison holding everywhere, for
/// the same reason as the nonnegativity above.
theorem partial_pointwise_le[S: OrderedField](f: Nat -> S, g: Nat -> S, n: Nat) {
    (forall(i: Nat) { f(i) <= g(i) }) implies partial(f, n) <= partial(g, n)
} by {
    if forall(i: Nat) { f(i) <= g(i) } {
        define p(x: Nat) -> Bool {
            partial(f, x) <= partial(g, x)
        }
        partial_zero(f)
        partial_zero(g)
        lte_refl(partial(g, Nat.0))
        partial(f, Nat.0) <= partial(g, Nat.0)
        p(Nat.0)
        forall(k: Nat) {
            if p(k) {
                partial(f, k) <= partial(g, k)
                f(k) <= g(k)
                partial(f, k) + f(k) <= partial(g, k) + g(k)
                partial_split_last(f, k)
                partial_split_last(g, k)
                partial(f, k.suc) <= partial(g, k.suc)
                p(k.suc)
            }
            (p(k) implies p(k.suc))
        }
        p(Nat.0) and forall(k: Nat) {
            p(k) implies p(k.suc)
        }
        Nat.induction(p)
        p(n)
        partial(f, n) <= partial(g, n)
    }
}

/// The zero extension is monotone in the function.
theorem fin_value_or_zero_le[S: OrderedField](
    n: Nat, f: Fin[n] -> S, g: Fin[n] -> S, k: Nat
) {
    (forall(i: Fin[n]) { f(i) <= g(i) })
        implies fin_value_or_zero[S](n, f, k) <= fin_value_or_zero[S](n, g, k)
} by {
    if forall(i: Fin[n]) { f(i) <= g(i) } {
        if k < n {
            let (i: Fin[n]) satisfy {
                Fin[n].new(k) = Option.some(i)
            }
            fin_value_or_zero_value[S](n, f, i)
            fin_value_or_zero_value[S](n, g, i)
            i.value = k
            fin_value_or_zero[S](n, f, k) = f(i)
            fin_value_or_zero[S](n, g, k) = g(i)
            f(i) <= g(i)
            fin_value_or_zero[S](n, f, k) <= fin_value_or_zero[S](n, g, k)
        }
        if not (k < n) {
            fin_value_or_zero_of_not_lt[S](n, f, k)
            fin_value_or_zero[S](n, f, k) = S.0
            fin_value_or_zero_of_not_lt[S](n, g, k)
            fin_value_or_zero[S](n, g, k) = S.0
            lte_refl(S.0)
            fin_value_or_zero[S](n, f, k) <= fin_value_or_zero[S](n, g, k)
        }
        fin_value_or_zero[S](n, f, k) <= fin_value_or_zero[S](n, g, k)
    }
}

/// Finite sums are monotone in the summand.
theorem fin_sum_mono[S: OrderedField](n: Nat, f: Fin[n] -> S, g: Fin[n] -> S) {
    (forall(i: Fin[n]) { f(i) <= g(i) }) implies fin_sum[S](n, f) <= fin_sum[S](n, g)
} by {
    if forall(i: Fin[n]) { f(i) <= g(i) } {
        forall(k: Nat) {
            fin_value_or_zero_le[S](n, f, g, k)
            fin_value_or_zero[S](n, f, k) <= fin_value_or_zero[S](n, g, k)
        }
        partial_pointwise_le[S](fin_value_or_zero[S](n, f), fin_value_or_zero[S](n, g), n)
        (partial(fin_value_or_zero[S](n, f), n) <= partial(fin_value_or_zero[S](n, g), n))
        fin_sum_apply[S](n, f)
        fin_sum_apply[S](n, g)
        fin_sum[S](n, f) <= fin_sum[S](n, g)
    }
}
