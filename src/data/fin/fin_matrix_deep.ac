/// General-n determinant deepening: the row and column multilinearity
/// (ported from the #1497 work), the alternating property at both the
/// determinant-list level and for the canonical determinant
/// (`matrix_det_suc_equal_rows`), matrix multiplication associativity, and the
/// characteristic-polynomial constant term.  These are the ingredients the
/// product theorem `det(A*B) = det(A)*det(B)`, the adjugate identity
/// `A * adj(A) = det(A) * I` and the inverse properties need; the remaining
/// gaps are recorded in the notes at the end of this file.
/// Implementation-internal: nothing here is exported through an interface.

from nat import Nat, strong_induction, true_below, true_below_apply, lt_and_lte,
    lte_and_lt, lte_cancel_suc, lt_cancel_suc, lt_suc, lt_not_ref, lte_antisymm, lt_add_left, lt_trans,
    add_suc_left, trichotomy
from data.fin.fin import Fin, fin_zero, fin_zero_new, fin_zero_value, fin_of_nat_suc,
    fin_of_nat_suc_new_of_lt, fin_last, fin_last_new, fin_last_value, fin_succ, fin_succ_value,
    ext
from data.fin.fin_enum import fin_enum_suc, fin_enum_suc_contains,
    fin_enum_suc_is_unique, fin_enum_suc_length
from data.fin.fin_matrix import Matrix, determinant_list, determinant_list_cons_rows,
    determinant_list_nil_rows, list_index_or_zero, list_index_or_zero_cons_self,
    list_index_or_zero_cons_other, matrix_det_list, matrix_det_list_cons_rows,
    matrix_entry, matrix_entry_one_diag, matrix_entry_one_off_diag, matrix_entry_transpose,
    matrix_entry_zero, matrix_entry_mul, matrix_mul, matrix_mul_term, matrix_one,
    matrix_swap_rows, matrix_entry_swap_rows, matrix_swap_rows_entry, matrix_transpose,
    matrix_zero, square_matrix_entry_function, matrix_ext, matrix_minor, matrix_entry_minor,
    matrix_cofactor_with, matrix_cofactor_with_eq, matrix_cofactor_sign, matrix_laplace_row,
    matrix_laplace_row_term, matrix_laplace_first_row, matrix_laplace_first_row_term,
    square_matrix_mul, square_matrix_mul_one_right, square_matrix_mul_one_left
from data.fin.fin_matrix_det import matrix_det_suc, matrix_det_suc_unfold,
    determinant_list_cons_rows_term, det_row_sum_term, determinant_list_pointwise,
    sum_map_add_fn, sum_map_scalar_mul, sum_map_sum_exchange, sum_map_zero_fn,
    sum_map_neg, determinant_list_two_rows, swap_term_pair_neg,
    determinant_list_swap_first_two_rows, determinant_list_zero_row_head,
    unique_cons_not_contains_head, det_two_row_term, sub_zero_right
from data.fin.fin_matrix_2x2 import char_poly_2x2, matrix2_00, matrix2_01, matrix2_10,
    matrix2_11, matrix2_det_entries, matrix2_trace
from data.fin.fin_sum import fin_sum
from data.fin.fin_sum_enum import fin_sum_suc_eq_sum_fin_enum_suc
from algebra.add_comm_monoid import AddCommMonoid
from algebra.add_semigroup import add_fn
from algebra.add_group import inverse_inverse
from semiring import Semiring
from comm_ring import CommRing
from list import List, map, sum, map_singleton, remove_one_cons_eq, remove_one_cons_neq,
    sum_map_of_pointwise, sum_map_remove_one, map_sum_add, unique_implies_tail_unique,
    remove_one_unique, remove_one_unique_not_contains_self, remove_one_contains_other
from algebra.ring.ring import Ring, alternating_sign, alternating_sign_zero,
    alternating_sign_suc, alternating_sign_suc_mul, mul_neg_left, mul_neg_right, mul_neg_neg, mul_neg_one_left,
    mul_neg_one_right

numerals Nat

// ---------------------------------------------------------------------------
// Row multilinearity of the determinant (ported from the #1497 work)
// ---------------------------------------------------------------------------
//
// A determinant is linear in each of its rows: replacing one row by the sum of
// two row functions splits the determinant into the sum of the two
// determinants, and replacing a row by a scalar multiple extracts the scalar.
// These are the core components of the product theorem `det(A*B) = det(A) *
// det(B)`.


/// The entry function obtained by replacing one row with a supplied row function.
define det_row_replace_entry[R: Ring, I](entry: (I, I) -> R, r: I, g: (I) -> R, i: I, c: I) -> R {
    if i = r {
        g(c)
    } else {
        entry(i, c)
    }
}

/// The pointwise sum of two row functions.
define det_row_fn_add[R: Ring, I](g1: (I) -> R, g2: (I) -> R, c: I) -> R {
    g1(c) + g2(c)
}

/// The pointwise scalar multiple of a row function.
define det_row_fn_smul[R: Ring, I](c: R, g: (I) -> R, i: I) -> R {
    c * g(i)
}

/// Replacing a row that is absent from the row list leaves a determinant-list unchanged.
theorem determinant_list_row_absent[R: Ring, I](entry: (I, I) -> R, g: (I) -> R, r: I, rows: List[I], cols: List[I]) {
    not rows.contains(r) implies
        determinant_list[R, I](det_row_replace_entry[R, I](entry, r, g), rows, cols) =
            determinant_list[R, I](entry, rows, cols)
} by {
    if not rows.contains(r) {
        define p(l: List[I]) -> Bool {
            not l.contains(r) implies
                forall(cols2: List[I]) {
                    determinant_list[R, I](det_row_replace_entry[R, I](entry, r, g), l, cols2) =
                        determinant_list[R, I](entry, l, cols2)
                }
        }
        if not List.nil[I].contains(r) {
            determinant_list_nil_rows[R, I](det_row_replace_entry[R, I](entry, r, g), List.nil[I])
            determinant_list[R, I](det_row_replace_entry[R, I](entry, r, g), List.nil[I], List.nil[I]) = R.1
            determinant_list_nil_rows[R, I](entry, List.nil[I])
            determinant_list[R, I](entry, List.nil[I], List.nil[I]) = R.1
            forall(cols2: List[I]) {
                determinant_list[R, I](det_row_replace_entry[R, I](entry, r, g), List.nil[I], cols2) =
                    determinant_list[R, I](entry, List.nil[I], cols2)
            }
        }
        p(List.nil[I])
        forall(head: I, tail: List[I]) {
            if p(tail) {
                if not List.cons(head, tail).contains(r) {
                    if head = r {
                        List.cons(head, tail).contains(r) = true
                        List.cons(head, tail).contains(r) = false
                        false
                    }
                    if head != r {
                        forall(cols2: List[I]) {
                            determinant_list_cons_rows_term[R, I](det_row_replace_entry[R, I](entry, r, g), head, tail, cols2)
                            determinant_list[R, I](det_row_replace_entry[R, I](entry, r, g), List.cons(head, tail), cols2) =
                                sum[R](map[I, R](cols2, det_row_sum_term[R, I](det_row_replace_entry[R, I](entry, r, g), head, tail, cols2)))
                            determinant_list_cons_rows_term[R, I](entry, head, tail, cols2)
                            determinant_list[R, I](entry, List.cons(head, tail), cols2) =
                                sum[R](map[I, R](cols2, det_row_sum_term[R, I](entry, head, tail, cols2)))
                            List.cons(head, tail).contains(r) = tail.contains(r)
                            not tail.contains(r)
                            p(tail) = (not tail.contains(r) implies
                                forall(cols3: List[I]) {
                                    determinant_list[R, I](det_row_replace_entry[R, I](entry, r, g), tail, cols3) =
                                        determinant_list[R, I](entry, tail, cols3)
                                })
                            forall(cols3: List[I]) {
                                determinant_list[R, I](det_row_replace_entry[R, I](entry, r, g), tail, cols3) =
                                    determinant_list[R, I](entry, tail, cols3)
                            }
                            forall(col: I) {
                                if cols2.contains(col) {
                                    det_row_replace_entry[R, I](entry, r, g, head, col) = entry(head, col)
                                    determinant_list[R, I](det_row_replace_entry[R, I](entry, r, g), tail, cols2.remove_one(col)) =
                                        determinant_list[R, I](entry, tail, cols2.remove_one(col))
                                    det_row_sum_term[R, I](det_row_replace_entry[R, I](entry, r, g), head, tail, cols2, col) =
                                        entry(head, col) * alternating_sign[R](list_index_or_zero[I](cols2, col)) *
                                            determinant_list[R, I](det_row_replace_entry[R, I](entry, r, g), tail, cols2.remove_one(col))
                                    det_row_sum_term[R, I](entry, head, tail, cols2, col) =
                                        entry(head, col) * alternating_sign[R](list_index_or_zero[I](cols2, col)) *
                                            determinant_list[R, I](entry, tail, cols2.remove_one(col))
                                    det_row_sum_term[R, I](det_row_replace_entry[R, I](entry, r, g), head, tail, cols2, col) =
                                        det_row_sum_term[R, I](entry, head, tail, cols2, col)
                                }
                            }
                            sum_map_of_pointwise[I, R](cols2,
                                det_row_sum_term[R, I](det_row_replace_entry[R, I](entry, r, g), head, tail, cols2),
                                det_row_sum_term[R, I](entry, head, tail, cols2))
                            sum[R](map[I, R](cols2, det_row_sum_term[R, I](det_row_replace_entry[R, I](entry, r, g), head, tail, cols2))) =
                                sum[R](map[I, R](cols2, det_row_sum_term[R, I](entry, head, tail, cols2)))
                            determinant_list[R, I](det_row_replace_entry[R, I](entry, r, g), List.cons(head, tail), cols2) =
                                determinant_list[R, I](entry, List.cons(head, tail), cols2)
                        }
                    }
                    forall(cols2: List[I]) {
                        determinant_list[R, I](det_row_replace_entry[R, I](entry, r, g), List.cons(head, tail), cols2) =
                            determinant_list[R, I](entry, List.cons(head, tail), cols2)
                    }
                }
                p(tail) implies p(List.cons(head, tail))
            }
        }
        p(List.nil[I]) and forall(head: I, tail: List[I]) { p(tail) implies p(List.cons(head, tail)) }
        List.induction(p)
        forall(l: List[I]) { p(l) }
        p(rows)
        forall(cols2: List[I]) {
            determinant_list[R, I](det_row_replace_entry[R, I](entry, r, g), rows, cols2) =
                determinant_list[R, I](entry, rows, cols2)
        }
        determinant_list[R, I](det_row_replace_entry[R, I](entry, r, g), rows, cols) =
            determinant_list[R, I](entry, rows, cols)
    }
}

/// A determinant-list is additive in a single row: replacing the row `r` of an
/// entry function by the sum of two row functions splits the determinant into
/// the sum of the two determinants, provided the row `r` occurs exactly once in
/// the row list.
theorem determinant_list_row_add[R: Ring, I](entry: (I, I) -> R, g1: (I) -> R, g2: (I) -> R, r: I, rows: List[I], cols: List[I]) {
    rows.contains(r) and not rows.remove_one(r).contains(r) implies
        determinant_list[R, I](det_row_replace_entry[R, I](entry, r, det_row_fn_add[R, I](g1, g2)), rows, cols) =
            determinant_list[R, I](det_row_replace_entry[R, I](entry, r, g1), rows, cols) +
                determinant_list[R, I](det_row_replace_entry[R, I](entry, r, g2), rows, cols)
} by {
    if rows.contains(r) and not rows.remove_one(r).contains(r) {
        define p(l: List[I]) -> Bool {
            l.contains(r) and not l.remove_one(r).contains(r) implies
                forall(cols2: List[I]) {
                    determinant_list[R, I](det_row_replace_entry[R, I](entry, r, det_row_fn_add[R, I](g1, g2)), l, cols2) =
                        determinant_list[R, I](det_row_replace_entry[R, I](entry, r, g1), l, cols2) +
                            determinant_list[R, I](det_row_replace_entry[R, I](entry, r, g2), l, cols2)
                }
        }
        if List.nil[I].contains(r) and not List.nil[I].remove_one(r).contains(r) {
            false
        }
        p(List.nil[I])
        forall(head: I, tail: List[I]) {
            if p(tail) {
                if List.cons(head, tail).contains(r) and not List.cons(head, tail).remove_one(r).contains(r) {
                    if head = r {
                        remove_one_cons_eq(head, tail)
                        List.cons(head, tail).remove_one(r) = tail
                        not tail.contains(r)
                        forall(cols2: List[I]) {
                            determinant_list_row_absent[R, I](entry, det_row_fn_add[R, I](g1, g2), r, tail, cols2)
                            determinant_list[R, I](det_row_replace_entry[R, I](entry, r, det_row_fn_add[R, I](g1, g2)), tail, cols2) =
                                determinant_list[R, I](entry, tail, cols2)
                            determinant_list_row_absent[R, I](entry, g1, r, tail, cols2)
                            determinant_list[R, I](det_row_replace_entry[R, I](entry, r, g1), tail, cols2) =
                                determinant_list[R, I](entry, tail, cols2)
                            determinant_list_row_absent[R, I](entry, g2, r, tail, cols2)
                            determinant_list[R, I](det_row_replace_entry[R, I](entry, r, g2), tail, cols2) =
                                determinant_list[R, I](entry, tail, cols2)
                            determinant_list[R, I](det_row_replace_entry[R, I](entry, r, det_row_fn_add[R, I](g1, g2)), tail, cols2) =
                                determinant_list[R, I](det_row_replace_entry[R, I](entry, r, g1), tail, cols2)
                            determinant_list[R, I](det_row_replace_entry[R, I](entry, r, det_row_fn_add[R, I](g1, g2)), tail, cols2) =
                                determinant_list[R, I](det_row_replace_entry[R, I](entry, r, g2), tail, cols2)
                            determinant_list_cons_rows_term[R, I](det_row_replace_entry[R, I](entry, r, det_row_fn_add[R, I](g1, g2)), head, tail, cols2)
                            determinant_list[R, I](det_row_replace_entry[R, I](entry, r, det_row_fn_add[R, I](g1, g2)), List.cons(head, tail), cols2) =
                                sum[R](map[I, R](cols2, det_row_sum_term[R, I](det_row_replace_entry[R, I](entry, r, det_row_fn_add[R, I](g1, g2)), head, tail, cols2)))
                            determinant_list_cons_rows_term[R, I](det_row_replace_entry[R, I](entry, r, g1), head, tail, cols2)
                            determinant_list[R, I](det_row_replace_entry[R, I](entry, r, g1), List.cons(head, tail), cols2) =
                                sum[R](map[I, R](cols2, det_row_sum_term[R, I](det_row_replace_entry[R, I](entry, r, g1), head, tail, cols2)))
                            determinant_list_cons_rows_term[R, I](det_row_replace_entry[R, I](entry, r, g2), head, tail, cols2)
                            determinant_list[R, I](det_row_replace_entry[R, I](entry, r, g2), List.cons(head, tail), cols2) =
                                sum[R](map[I, R](cols2, det_row_sum_term[R, I](det_row_replace_entry[R, I](entry, r, g2), head, tail, cols2)))
                            forall(col: I) {
                                if cols2.contains(col) {
                                    det_row_sum_term[R, I](det_row_replace_entry[R, I](entry, r, det_row_fn_add[R, I](g1, g2)), head, tail, cols2, col) =
                                        det_row_replace_entry[R, I](entry, r, det_row_fn_add[R, I](g1, g2), head, col) *
                                            alternating_sign[R](list_index_or_zero[I](cols2, col)) *
                                            determinant_list[R, I](det_row_replace_entry[R, I](entry, r, det_row_fn_add[R, I](g1, g2)), tail, cols2.remove_one(col))
                                    det_row_replace_entry[R, I](entry, r, det_row_fn_add[R, I](g1, g2), head, col) = det_row_fn_add[R, I](g1, g2, col)
                                    det_row_fn_add[R, I](g1, g2, col) = g1(col) + g2(col)
                                    det_row_replace_entry[R, I](entry, r, det_row_fn_add[R, I](g1, g2), head, col) = g1(col) + g2(col)
                                    determinant_list_row_absent[R, I](entry, det_row_fn_add[R, I](g1, g2), r, tail, cols2.remove_one(col))
                                    determinant_list[R, I](det_row_replace_entry[R, I](entry, r, det_row_fn_add[R, I](g1, g2)), tail, cols2.remove_one(col)) =
                                        determinant_list[R, I](entry, tail, cols2.remove_one(col))
                                    determinant_list_row_absent[R, I](entry, g1, r, tail, cols2.remove_one(col))
                                    determinant_list[R, I](det_row_replace_entry[R, I](entry, r, g1), tail, cols2.remove_one(col)) =
                                        determinant_list[R, I](entry, tail, cols2.remove_one(col))
                                    determinant_list_row_absent[R, I](entry, g2, r, tail, cols2.remove_one(col))
                                    determinant_list[R, I](det_row_replace_entry[R, I](entry, r, g2), tail, cols2.remove_one(col)) =
                                        determinant_list[R, I](entry, tail, cols2.remove_one(col))
                                    determinant_list[R, I](det_row_replace_entry[R, I](entry, r, det_row_fn_add[R, I](g1, g2)), tail, cols2.remove_one(col)) =
                                        determinant_list[R, I](det_row_replace_entry[R, I](entry, r, g1), tail, cols2.remove_one(col))
                                    det_row_sum_term[R, I](det_row_replace_entry[R, I](entry, r, det_row_fn_add[R, I](g1, g2)), head, tail, cols2, col) =
                                        (g1(col) + g2(col)) * alternating_sign[R](list_index_or_zero[I](cols2, col)) *
                                            determinant_list[R, I](det_row_replace_entry[R, I](entry, r, g1), tail, cols2.remove_one(col))
                                    (g1(col) + g2(col)) * alternating_sign[R](list_index_or_zero[I](cols2, col)) *
                                        determinant_list[R, I](det_row_replace_entry[R, I](entry, r, g1), tail, cols2.remove_one(col)) =
                                        g1(col) * alternating_sign[R](list_index_or_zero[I](cols2, col)) *
                                            determinant_list[R, I](det_row_replace_entry[R, I](entry, r, g1), tail, cols2.remove_one(col)) +
                                        g2(col) * alternating_sign[R](list_index_or_zero[I](cols2, col)) *
                                            determinant_list[R, I](det_row_replace_entry[R, I](entry, r, g1), tail, cols2.remove_one(col))
                                    det_row_sum_term[R, I](det_row_replace_entry[R, I](entry, r, g1), head, tail, cols2, col) =
                                        det_row_replace_entry[R, I](entry, r, g1, head, col) *
                                            alternating_sign[R](list_index_or_zero[I](cols2, col)) *
                                            determinant_list[R, I](det_row_replace_entry[R, I](entry, r, g1), tail, cols2.remove_one(col))
                                    det_row_replace_entry[R, I](entry, r, g1, head, col) = g1(col)
                                    det_row_sum_term[R, I](det_row_replace_entry[R, I](entry, r, g1), head, tail, cols2, col) =
                                        g1(col) * alternating_sign[R](list_index_or_zero[I](cols2, col)) *
                                            determinant_list[R, I](det_row_replace_entry[R, I](entry, r, g1), tail, cols2.remove_one(col))
                                    det_row_sum_term[R, I](det_row_replace_entry[R, I](entry, r, g2), head, tail, cols2, col) =
                                        det_row_replace_entry[R, I](entry, r, g2, head, col) *
                                            alternating_sign[R](list_index_or_zero[I](cols2, col)) *
                                            determinant_list[R, I](det_row_replace_entry[R, I](entry, r, g2), tail, cols2.remove_one(col))
                                    det_row_replace_entry[R, I](entry, r, g2, head, col) = g2(col)
                                    determinant_list_row_absent[R, I](entry, g2, r, tail, cols2.remove_one(col))
                                    determinant_list[R, I](det_row_replace_entry[R, I](entry, r, g2), tail, cols2.remove_one(col)) =
                                        determinant_list[R, I](entry, tail, cols2.remove_one(col))
                                    determinant_list_row_absent[R, I](entry, g1, r, tail, cols2.remove_one(col))
                                    determinant_list[R, I](det_row_replace_entry[R, I](entry, r, g1), tail, cols2.remove_one(col)) =
                                        determinant_list[R, I](entry, tail, cols2.remove_one(col))
                                    determinant_list[R, I](det_row_replace_entry[R, I](entry, r, g2), tail, cols2.remove_one(col)) =
                                        determinant_list[R, I](det_row_replace_entry[R, I](entry, r, g1), tail, cols2.remove_one(col))
                                    g2(col) * alternating_sign[R](list_index_or_zero[I](cols2, col)) *
                                        determinant_list[R, I](det_row_replace_entry[R, I](entry, r, g2), tail, cols2.remove_one(col)) =
                                        g2(col) * alternating_sign[R](list_index_or_zero[I](cols2, col)) *
                                            determinant_list[R, I](det_row_replace_entry[R, I](entry, r, g1), tail, cols2.remove_one(col))
                                    det_row_sum_term[R, I](det_row_replace_entry[R, I](entry, r, g2), head, tail, cols2, col) =
                                        g2(col) * alternating_sign[R](list_index_or_zero[I](cols2, col)) *
                                            determinant_list[R, I](det_row_replace_entry[R, I](entry, r, g1), tail, cols2.remove_one(col))
                                    det_row_sum_term[R, I](det_row_replace_entry[R, I](entry, r, det_row_fn_add[R, I](g1, g2)), head, tail, cols2, col) =
                                        det_row_sum_term[R, I](det_row_replace_entry[R, I](entry, r, g1), head, tail, cols2, col) +
                                            det_row_sum_term[R, I](det_row_replace_entry[R, I](entry, r, g2), head, tail, cols2, col)
                                }
                            }
                            sum_map_of_pointwise[I, R](cols2,
                                det_row_sum_term[R, I](det_row_replace_entry[R, I](entry, r, det_row_fn_add[R, I](g1, g2)), head, tail, cols2),
                                function(col: I) {
                                    det_row_sum_term[R, I](det_row_replace_entry[R, I](entry, r, g1), head, tail, cols2, col) +
                                        det_row_sum_term[R, I](det_row_replace_entry[R, I](entry, r, g2), head, tail, cols2, col)
                                })
                            sum[R](map[I, R](cols2, det_row_sum_term[R, I](det_row_replace_entry[R, I](entry, r, det_row_fn_add[R, I](g1, g2)), head, tail, cols2))) =
                                sum[R](map[I, R](cols2, function(col: I) {
                                    det_row_sum_term[R, I](det_row_replace_entry[R, I](entry, r, g1), head, tail, cols2, col) +
                                        det_row_sum_term[R, I](det_row_replace_entry[R, I](entry, r, g2), head, tail, cols2, col)
                                }))
                            sum_map_add_fn[I, R](cols2,
                                det_row_sum_term[R, I](det_row_replace_entry[R, I](entry, r, g1), head, tail, cols2),
                                det_row_sum_term[R, I](det_row_replace_entry[R, I](entry, r, g2), head, tail, cols2))
                            sum[R](map[I, R](cols2, function(col: I) {
                                det_row_sum_term[R, I](det_row_replace_entry[R, I](entry, r, g1), head, tail, cols2, col) +
                                    det_row_sum_term[R, I](det_row_replace_entry[R, I](entry, r, g2), head, tail, cols2, col)
                            })) =
                                sum[R](map[I, R](cols2, det_row_sum_term[R, I](det_row_replace_entry[R, I](entry, r, g1), head, tail, cols2))) +
                                    sum[R](map[I, R](cols2, det_row_sum_term[R, I](det_row_replace_entry[R, I](entry, r, g2), head, tail, cols2)))
                            sum[R](map[I, R](cols2, det_row_sum_term[R, I](det_row_replace_entry[R, I](entry, r, det_row_fn_add[R, I](g1, g2)), head, tail, cols2))) =
                                sum[R](map[I, R](cols2, det_row_sum_term[R, I](det_row_replace_entry[R, I](entry, r, g1), head, tail, cols2))) +
                                    sum[R](map[I, R](cols2, det_row_sum_term[R, I](det_row_replace_entry[R, I](entry, r, g2), head, tail, cols2)))
                            determinant_list[R, I](det_row_replace_entry[R, I](entry, r, det_row_fn_add[R, I](g1, g2)), List.cons(head, tail), cols2) =
                                determinant_list[R, I](det_row_replace_entry[R, I](entry, r, g1), List.cons(head, tail), cols2) +
                                    determinant_list[R, I](det_row_replace_entry[R, I](entry, r, g2), List.cons(head, tail), cols2)
                        }
                    }
                    if head != r {
                        remove_one_cons_neq(head, tail, r)
                        List.cons(head, tail).remove_one(r) = List.cons(head, tail.remove_one(r))
                        tail.contains(r)
                        not List.cons(head, tail.remove_one(r)).contains(r)
                        not tail.remove_one(r).contains(r)
                        p(tail) = (tail.contains(r) and not tail.remove_one(r).contains(r) implies
                            forall(cols3: List[I]) {
                                determinant_list[R, I](det_row_replace_entry[R, I](entry, r, det_row_fn_add[R, I](g1, g2)), tail, cols3) =
                                    determinant_list[R, I](det_row_replace_entry[R, I](entry, r, g1), tail, cols3) +
                                        determinant_list[R, I](det_row_replace_entry[R, I](entry, r, g2), tail, cols3)
                            })
                        forall(cols3: List[I]) {
                            determinant_list[R, I](det_row_replace_entry[R, I](entry, r, det_row_fn_add[R, I](g1, g2)), tail, cols3) =
                                determinant_list[R, I](det_row_replace_entry[R, I](entry, r, g1), tail, cols3) +
                                    determinant_list[R, I](det_row_replace_entry[R, I](entry, r, g2), tail, cols3)
                        }
                        forall(cols2: List[I]) {
                            determinant_list_cons_rows_term[R, I](det_row_replace_entry[R, I](entry, r, det_row_fn_add[R, I](g1, g2)), head, tail, cols2)
                            determinant_list[R, I](det_row_replace_entry[R, I](entry, r, det_row_fn_add[R, I](g1, g2)), List.cons(head, tail), cols2) =
                                sum[R](map[I, R](cols2, det_row_sum_term[R, I](det_row_replace_entry[R, I](entry, r, det_row_fn_add[R, I](g1, g2)), head, tail, cols2)))
                            determinant_list_cons_rows_term[R, I](det_row_replace_entry[R, I](entry, r, g1), head, tail, cols2)
                            determinant_list[R, I](det_row_replace_entry[R, I](entry, r, g1), List.cons(head, tail), cols2) =
                                sum[R](map[I, R](cols2, det_row_sum_term[R, I](det_row_replace_entry[R, I](entry, r, g1), head, tail, cols2)))
                            determinant_list_cons_rows_term[R, I](det_row_replace_entry[R, I](entry, r, g2), head, tail, cols2)
                            determinant_list[R, I](det_row_replace_entry[R, I](entry, r, g2), List.cons(head, tail), cols2) =
                                sum[R](map[I, R](cols2, det_row_sum_term[R, I](det_row_replace_entry[R, I](entry, r, g2), head, tail, cols2)))
                            forall(col: I) {
                                if cols2.contains(col) {
                                    det_row_sum_term[R, I](det_row_replace_entry[R, I](entry, r, det_row_fn_add[R, I](g1, g2)), head, tail, cols2, col) =
                                        det_row_replace_entry[R, I](entry, r, det_row_fn_add[R, I](g1, g2), head, col) *
                                            alternating_sign[R](list_index_or_zero[I](cols2, col)) *
                                            determinant_list[R, I](det_row_replace_entry[R, I](entry, r, det_row_fn_add[R, I](g1, g2)), tail, cols2.remove_one(col))
                                    det_row_replace_entry[R, I](entry, r, det_row_fn_add[R, I](g1, g2), head, col) = entry(head, col)
                                    determinant_list[R, I](det_row_replace_entry[R, I](entry, r, det_row_fn_add[R, I](g1, g2)), tail, cols2.remove_one(col)) =
                                        determinant_list[R, I](det_row_replace_entry[R, I](entry, r, g1), tail, cols2.remove_one(col)) +
                                            determinant_list[R, I](det_row_replace_entry[R, I](entry, r, g2), tail, cols2.remove_one(col))
                                    det_row_sum_term[R, I](det_row_replace_entry[R, I](entry, r, det_row_fn_add[R, I](g1, g2)), head, tail, cols2, col) =
                                        entry(head, col) * alternating_sign[R](list_index_or_zero[I](cols2, col)) *
                                            (determinant_list[R, I](det_row_replace_entry[R, I](entry, r, g1), tail, cols2.remove_one(col)) +
                                                determinant_list[R, I](det_row_replace_entry[R, I](entry, r, g2), tail, cols2.remove_one(col)))
                                    entry(head, col) * alternating_sign[R](list_index_or_zero[I](cols2, col)) *
                                        (determinant_list[R, I](det_row_replace_entry[R, I](entry, r, g1), tail, cols2.remove_one(col)) +
                                            determinant_list[R, I](det_row_replace_entry[R, I](entry, r, g2), tail, cols2.remove_one(col))) =
                                        entry(head, col) * alternating_sign[R](list_index_or_zero[I](cols2, col)) *
                                            determinant_list[R, I](det_row_replace_entry[R, I](entry, r, g1), tail, cols2.remove_one(col)) +
                                        entry(head, col) * alternating_sign[R](list_index_or_zero[I](cols2, col)) *
                                            determinant_list[R, I](det_row_replace_entry[R, I](entry, r, g2), tail, cols2.remove_one(col))
                                    det_row_sum_term[R, I](det_row_replace_entry[R, I](entry, r, g1), head, tail, cols2, col) =
                                        det_row_replace_entry[R, I](entry, r, g1, head, col) *
                                            alternating_sign[R](list_index_or_zero[I](cols2, col)) *
                                            determinant_list[R, I](det_row_replace_entry[R, I](entry, r, g1), tail, cols2.remove_one(col))
                                    det_row_replace_entry[R, I](entry, r, g1, head, col) = entry(head, col)
                                    det_row_sum_term[R, I](det_row_replace_entry[R, I](entry, r, g1), head, tail, cols2, col) =
                                        entry(head, col) * alternating_sign[R](list_index_or_zero[I](cols2, col)) *
                                            determinant_list[R, I](det_row_replace_entry[R, I](entry, r, g1), tail, cols2.remove_one(col))
                                    det_row_sum_term[R, I](det_row_replace_entry[R, I](entry, r, g2), head, tail, cols2, col) =
                                        det_row_replace_entry[R, I](entry, r, g2, head, col) *
                                            alternating_sign[R](list_index_or_zero[I](cols2, col)) *
                                            determinant_list[R, I](det_row_replace_entry[R, I](entry, r, g2), tail, cols2.remove_one(col))
                                    det_row_replace_entry[R, I](entry, r, g2, head, col) = entry(head, col)
                                    det_row_sum_term[R, I](det_row_replace_entry[R, I](entry, r, g2), head, tail, cols2, col) =
                                        entry(head, col) * alternating_sign[R](list_index_or_zero[I](cols2, col)) *
                                            determinant_list[R, I](det_row_replace_entry[R, I](entry, r, g2), tail, cols2.remove_one(col))
                                    det_row_sum_term[R, I](det_row_replace_entry[R, I](entry, r, det_row_fn_add[R, I](g1, g2)), head, tail, cols2, col) =
                                        det_row_sum_term[R, I](det_row_replace_entry[R, I](entry, r, g1), head, tail, cols2, col) +
                                            det_row_sum_term[R, I](det_row_replace_entry[R, I](entry, r, g2), head, tail, cols2, col)
                                }
                            }
                            sum_map_of_pointwise[I, R](cols2,
                                det_row_sum_term[R, I](det_row_replace_entry[R, I](entry, r, det_row_fn_add[R, I](g1, g2)), head, tail, cols2),
                                function(col: I) {
                                    det_row_sum_term[R, I](det_row_replace_entry[R, I](entry, r, g1), head, tail, cols2, col) +
                                        det_row_sum_term[R, I](det_row_replace_entry[R, I](entry, r, g2), head, tail, cols2, col)
                                })
                            sum[R](map[I, R](cols2, det_row_sum_term[R, I](det_row_replace_entry[R, I](entry, r, det_row_fn_add[R, I](g1, g2)), head, tail, cols2))) =
                                sum[R](map[I, R](cols2, function(col: I) {
                                    det_row_sum_term[R, I](det_row_replace_entry[R, I](entry, r, g1), head, tail, cols2, col) +
                                        det_row_sum_term[R, I](det_row_replace_entry[R, I](entry, r, g2), head, tail, cols2, col)
                                }))
                            sum_map_add_fn[I, R](cols2,
                                det_row_sum_term[R, I](det_row_replace_entry[R, I](entry, r, g1), head, tail, cols2),
                                det_row_sum_term[R, I](det_row_replace_entry[R, I](entry, r, g2), head, tail, cols2))
                            sum[R](map[I, R](cols2, function(col: I) {
                                det_row_sum_term[R, I](det_row_replace_entry[R, I](entry, r, g1), head, tail, cols2, col) +
                                    det_row_sum_term[R, I](det_row_replace_entry[R, I](entry, r, g2), head, tail, cols2, col)
                            })) =
                                sum[R](map[I, R](cols2, det_row_sum_term[R, I](det_row_replace_entry[R, I](entry, r, g1), head, tail, cols2))) +
                                    sum[R](map[I, R](cols2, det_row_sum_term[R, I](det_row_replace_entry[R, I](entry, r, g2), head, tail, cols2)))
                            sum[R](map[I, R](cols2, det_row_sum_term[R, I](det_row_replace_entry[R, I](entry, r, det_row_fn_add[R, I](g1, g2)), head, tail, cols2))) =
                                sum[R](map[I, R](cols2, det_row_sum_term[R, I](det_row_replace_entry[R, I](entry, r, g1), head, tail, cols2))) +
                                    sum[R](map[I, R](cols2, det_row_sum_term[R, I](det_row_replace_entry[R, I](entry, r, g2), head, tail, cols2)))
                            determinant_list[R, I](det_row_replace_entry[R, I](entry, r, det_row_fn_add[R, I](g1, g2)), List.cons(head, tail), cols2) =
                                determinant_list[R, I](det_row_replace_entry[R, I](entry, r, g1), List.cons(head, tail), cols2) +
                                    determinant_list[R, I](det_row_replace_entry[R, I](entry, r, g2), List.cons(head, tail), cols2)
                        }
                    }
                    forall(cols2: List[I]) {
                        determinant_list[R, I](det_row_replace_entry[R, I](entry, r, det_row_fn_add[R, I](g1, g2)), List.cons(head, tail), cols2) =
                            determinant_list[R, I](det_row_replace_entry[R, I](entry, r, g1), List.cons(head, tail), cols2) +
                                determinant_list[R, I](det_row_replace_entry[R, I](entry, r, g2), List.cons(head, tail), cols2)
                    }
                }
                p(tail) implies p(List.cons(head, tail))
            }
        }
        p(List.nil[I]) and forall(head: I, tail: List[I]) { p(tail) implies p(List.cons(head, tail)) }
        List.induction(p)
        forall(l: List[I]) { p(l) }
        p(rows)
        forall(cols2: List[I]) {
            determinant_list[R, I](det_row_replace_entry[R, I](entry, r, det_row_fn_add[R, I](g1, g2)), rows, cols2) =
                determinant_list[R, I](det_row_replace_entry[R, I](entry, r, g1), rows, cols2) +
                    determinant_list[R, I](det_row_replace_entry[R, I](entry, r, g2), rows, cols2)
        }
        determinant_list[R, I](det_row_replace_entry[R, I](entry, r, det_row_fn_add[R, I](g1, g2)), rows, cols) =
            determinant_list[R, I](det_row_replace_entry[R, I](entry, r, g1), rows, cols) +
                determinant_list[R, I](det_row_replace_entry[R, I](entry, r, g2), rows, cols)
    }
}

/// A determinant-list is homogeneous in a single row: replacing the row `r` of
/// an entry function by a scalar multiple of a row function multiplies the
/// determinant by the scalar, provided the row `r` occurs exactly once in the
/// row list.
theorem determinant_list_row_smul[R: CommRing, I](entry: (I, I) -> R, c: R, g: (I) -> R, r: I, rows: List[I], cols: List[I]) {
    rows.contains(r) and not rows.remove_one(r).contains(r) implies
        determinant_list[R, I](det_row_replace_entry[R, I](entry, r, det_row_fn_smul[R, I](c, g)), rows, cols) =
            c * determinant_list[R, I](det_row_replace_entry[R, I](entry, r, g), rows, cols)
} by {
    if rows.contains(r) and not rows.remove_one(r).contains(r) {
        define p(l: List[I]) -> Bool {
            l.contains(r) and not l.remove_one(r).contains(r) implies
                forall(cols2: List[I]) {
                    determinant_list[R, I](det_row_replace_entry[R, I](entry, r, det_row_fn_smul[R, I](c, g)), l, cols2) =
                        c * determinant_list[R, I](det_row_replace_entry[R, I](entry, r, g), l, cols2)
                }
        }
        if List.nil[I].contains(r) and not List.nil[I].remove_one(r).contains(r) {
            false
        }
        p(List.nil[I])
        forall(head: I, tail: List[I]) {
            if p(tail) {
                if List.cons(head, tail).contains(r) and not List.cons(head, tail).remove_one(r).contains(r) {
                    if head = r {
                        remove_one_cons_eq(head, tail)
                        List.cons(head, tail).remove_one(r) = tail
                        not tail.contains(r)
                        forall(cols2: List[I]) {
                            determinant_list_row_absent[R, I](entry, det_row_fn_smul[R, I](c, g), r, tail, cols2)
                            determinant_list[R, I](det_row_replace_entry[R, I](entry, r, det_row_fn_smul[R, I](c, g)), tail, cols2) =
                                determinant_list[R, I](entry, tail, cols2)
                            determinant_list_row_absent[R, I](entry, g, r, tail, cols2)
                            determinant_list[R, I](det_row_replace_entry[R, I](entry, r, g), tail, cols2) =
                                determinant_list[R, I](entry, tail, cols2)
                            determinant_list[R, I](det_row_replace_entry[R, I](entry, r, det_row_fn_smul[R, I](c, g)), tail, cols2) =
                                determinant_list[R, I](det_row_replace_entry[R, I](entry, r, g), tail, cols2)
                            determinant_list_cons_rows_term[R, I](det_row_replace_entry[R, I](entry, r, det_row_fn_smul[R, I](c, g)), head, tail, cols2)
                            determinant_list[R, I](det_row_replace_entry[R, I](entry, r, det_row_fn_smul[R, I](c, g)), List.cons(head, tail), cols2) =
                                sum[R](map[I, R](cols2, det_row_sum_term[R, I](det_row_replace_entry[R, I](entry, r, det_row_fn_smul[R, I](c, g)), head, tail, cols2)))
                            determinant_list_cons_rows_term[R, I](det_row_replace_entry[R, I](entry, r, g), head, tail, cols2)
                            determinant_list[R, I](det_row_replace_entry[R, I](entry, r, g), List.cons(head, tail), cols2) =
                                sum[R](map[I, R](cols2, det_row_sum_term[R, I](det_row_replace_entry[R, I](entry, r, g), head, tail, cols2)))
                            forall(col: I) {
                                if cols2.contains(col) {
                                    det_row_sum_term[R, I](det_row_replace_entry[R, I](entry, r, det_row_fn_smul[R, I](c, g)), head, tail, cols2, col) =
                                        det_row_replace_entry[R, I](entry, r, det_row_fn_smul[R, I](c, g), head, col) *
                                            alternating_sign[R](list_index_or_zero[I](cols2, col)) *
                                            determinant_list[R, I](det_row_replace_entry[R, I](entry, r, det_row_fn_smul[R, I](c, g)), tail, cols2.remove_one(col))
                                    det_row_replace_entry[R, I](entry, r, det_row_fn_smul[R, I](c, g), head, col) = det_row_fn_smul[R, I](c, g, col)
                                    det_row_fn_smul[R, I](c, g, col) = c * g(col)
                                    det_row_replace_entry[R, I](entry, r, det_row_fn_smul[R, I](c, g), head, col) = c * g(col)
                                    determinant_list_row_absent[R, I](entry, g, r, tail, cols2.remove_one(col))
                                    determinant_list[R, I](det_row_replace_entry[R, I](entry, r, g), tail, cols2.remove_one(col)) =
                                        determinant_list[R, I](entry, tail, cols2.remove_one(col))
                                    determinant_list_row_absent[R, I](entry, det_row_fn_smul[R, I](c, g), r, tail, cols2.remove_one(col))
                                    determinant_list[R, I](det_row_replace_entry[R, I](entry, r, det_row_fn_smul[R, I](c, g)), tail, cols2.remove_one(col)) =
                                        determinant_list[R, I](entry, tail, cols2.remove_one(col))
                                    det_row_sum_term[R, I](det_row_replace_entry[R, I](entry, r, det_row_fn_smul[R, I](c, g)), head, tail, cols2, col) =
                                        (c * g(col)) * alternating_sign[R](list_index_or_zero[I](cols2, col)) *
                                            determinant_list[R, I](det_row_replace_entry[R, I](entry, r, g), tail, cols2.remove_one(col))
                                    (c * g(col)) * alternating_sign[R](list_index_or_zero[I](cols2, col)) *
                                        determinant_list[R, I](det_row_replace_entry[R, I](entry, r, g), tail, cols2.remove_one(col)) =
                                        c * (g(col) * alternating_sign[R](list_index_or_zero[I](cols2, col)) *
                                            determinant_list[R, I](det_row_replace_entry[R, I](entry, r, g), tail, cols2.remove_one(col)))
                                    det_row_sum_term[R, I](det_row_replace_entry[R, I](entry, r, g), head, tail, cols2, col) =
                                        det_row_replace_entry[R, I](entry, r, g, head, col) *
                                            alternating_sign[R](list_index_or_zero[I](cols2, col)) *
                                            determinant_list[R, I](det_row_replace_entry[R, I](entry, r, g), tail, cols2.remove_one(col))
                                    det_row_replace_entry[R, I](entry, r, g, head, col) = g(col)
                                    det_row_sum_term[R, I](det_row_replace_entry[R, I](entry, r, g), head, tail, cols2, col) =
                                        g(col) * alternating_sign[R](list_index_or_zero[I](cols2, col)) *
                                            determinant_list[R, I](det_row_replace_entry[R, I](entry, r, g), tail, cols2.remove_one(col))
                                    det_row_sum_term[R, I](det_row_replace_entry[R, I](entry, r, det_row_fn_smul[R, I](c, g)), head, tail, cols2, col) =
                                        c * det_row_sum_term[R, I](det_row_replace_entry[R, I](entry, r, g), head, tail, cols2, col)
                                }
                            }
                            sum_map_of_pointwise[I, R](cols2,
                                det_row_sum_term[R, I](det_row_replace_entry[R, I](entry, r, det_row_fn_smul[R, I](c, g)), head, tail, cols2),
                                function(col: I) {
                                    c * det_row_sum_term[R, I](det_row_replace_entry[R, I](entry, r, g), head, tail, cols2, col)
                                })
                            sum[R](map[I, R](cols2, det_row_sum_term[R, I](det_row_replace_entry[R, I](entry, r, det_row_fn_smul[R, I](c, g)), head, tail, cols2))) =
                                sum[R](map[I, R](cols2, function(col: I) {
                                    c * det_row_sum_term[R, I](det_row_replace_entry[R, I](entry, r, g), head, tail, cols2, col)
                                }))
                            sum_map_scalar_mul[I, R](cols2, c, det_row_sum_term[R, I](det_row_replace_entry[R, I](entry, r, g), head, tail, cols2))
                            c * sum[R](map[I, R](cols2, det_row_sum_term[R, I](det_row_replace_entry[R, I](entry, r, g), head, tail, cols2))) =
                                sum[R](map[I, R](cols2, function(col: I) {
                                    c * det_row_sum_term[R, I](det_row_replace_entry[R, I](entry, r, g), head, tail, cols2, col)
                                }))
                            sum[R](map[I, R](cols2, det_row_sum_term[R, I](det_row_replace_entry[R, I](entry, r, det_row_fn_smul[R, I](c, g)), head, tail, cols2))) =
                                c * sum[R](map[I, R](cols2, det_row_sum_term[R, I](det_row_replace_entry[R, I](entry, r, g), head, tail, cols2)))
                            determinant_list[R, I](det_row_replace_entry[R, I](entry, r, det_row_fn_smul[R, I](c, g)), List.cons(head, tail), cols2) =
                                c * determinant_list[R, I](det_row_replace_entry[R, I](entry, r, g), List.cons(head, tail), cols2)
                        }
                    }
                    if head != r {
                        remove_one_cons_neq(head, tail, r)
                        List.cons(head, tail).remove_one(r) = List.cons(head, tail.remove_one(r))
                        tail.contains(r)
                        not List.cons(head, tail.remove_one(r)).contains(r)
                        not tail.remove_one(r).contains(r)
                        p(tail) = (tail.contains(r) and not tail.remove_one(r).contains(r) implies
                            forall(cols3: List[I]) {
                                determinant_list[R, I](det_row_replace_entry[R, I](entry, r, det_row_fn_smul[R, I](c, g)), tail, cols3) =
                                    c * determinant_list[R, I](det_row_replace_entry[R, I](entry, r, g), tail, cols3)
                            })
                        forall(cols3: List[I]) {
                            determinant_list[R, I](det_row_replace_entry[R, I](entry, r, det_row_fn_smul[R, I](c, g)), tail, cols3) =
                                c * determinant_list[R, I](det_row_replace_entry[R, I](entry, r, g), tail, cols3)
                        }
                        forall(cols2: List[I]) {
                            determinant_list_cons_rows_term[R, I](det_row_replace_entry[R, I](entry, r, det_row_fn_smul[R, I](c, g)), head, tail, cols2)
                            determinant_list[R, I](det_row_replace_entry[R, I](entry, r, det_row_fn_smul[R, I](c, g)), List.cons(head, tail), cols2) =
                                sum[R](map[I, R](cols2, det_row_sum_term[R, I](det_row_replace_entry[R, I](entry, r, det_row_fn_smul[R, I](c, g)), head, tail, cols2)))
                            determinant_list_cons_rows_term[R, I](det_row_replace_entry[R, I](entry, r, g), head, tail, cols2)
                            determinant_list[R, I](det_row_replace_entry[R, I](entry, r, g), List.cons(head, tail), cols2) =
                                sum[R](map[I, R](cols2, det_row_sum_term[R, I](det_row_replace_entry[R, I](entry, r, g), head, tail, cols2)))
                            forall(col: I) {
                                if cols2.contains(col) {
                                    det_row_sum_term[R, I](det_row_replace_entry[R, I](entry, r, det_row_fn_smul[R, I](c, g)), head, tail, cols2, col) =
                                        det_row_replace_entry[R, I](entry, r, det_row_fn_smul[R, I](c, g), head, col) *
                                            alternating_sign[R](list_index_or_zero[I](cols2, col)) *
                                            determinant_list[R, I](det_row_replace_entry[R, I](entry, r, det_row_fn_smul[R, I](c, g)), tail, cols2.remove_one(col))
                                    det_row_replace_entry[R, I](entry, r, det_row_fn_smul[R, I](c, g), head, col) = entry(head, col)
                                    determinant_list[R, I](det_row_replace_entry[R, I](entry, r, det_row_fn_smul[R, I](c, g)), tail, cols2.remove_one(col)) =
                                        c * determinant_list[R, I](det_row_replace_entry[R, I](entry, r, g), tail, cols2.remove_one(col))
                                    det_row_sum_term[R, I](det_row_replace_entry[R, I](entry, r, det_row_fn_smul[R, I](c, g)), head, tail, cols2, col) =
                                        entry(head, col) * alternating_sign[R](list_index_or_zero[I](cols2, col)) *
                                            (c * determinant_list[R, I](det_row_replace_entry[R, I](entry, r, g), tail, cols2.remove_one(col)))
                                    entry(head, col) * alternating_sign[R](list_index_or_zero[I](cols2, col)) *
                                        (c * determinant_list[R, I](det_row_replace_entry[R, I](entry, r, g), tail, cols2.remove_one(col))) =
                                        c * (entry(head, col) * alternating_sign[R](list_index_or_zero[I](cols2, col)) *
                                            determinant_list[R, I](det_row_replace_entry[R, I](entry, r, g), tail, cols2.remove_one(col)))
                                    det_row_sum_term[R, I](det_row_replace_entry[R, I](entry, r, g), head, tail, cols2, col) =
                                        det_row_replace_entry[R, I](entry, r, g, head, col) *
                                            alternating_sign[R](list_index_or_zero[I](cols2, col)) *
                                            determinant_list[R, I](det_row_replace_entry[R, I](entry, r, g), tail, cols2.remove_one(col))
                                    det_row_replace_entry[R, I](entry, r, g, head, col) = entry(head, col)
                                    det_row_sum_term[R, I](det_row_replace_entry[R, I](entry, r, g), head, tail, cols2, col) =
                                        entry(head, col) * alternating_sign[R](list_index_or_zero[I](cols2, col)) *
                                            determinant_list[R, I](det_row_replace_entry[R, I](entry, r, g), tail, cols2.remove_one(col))
                                    det_row_sum_term[R, I](det_row_replace_entry[R, I](entry, r, det_row_fn_smul[R, I](c, g)), head, tail, cols2, col) =
                                        c * det_row_sum_term[R, I](det_row_replace_entry[R, I](entry, r, g), head, tail, cols2, col)
                                }
                            }
                            sum_map_of_pointwise[I, R](cols2,
                                det_row_sum_term[R, I](det_row_replace_entry[R, I](entry, r, det_row_fn_smul[R, I](c, g)), head, tail, cols2),
                                function(col: I) {
                                    c * det_row_sum_term[R, I](det_row_replace_entry[R, I](entry, r, g), head, tail, cols2, col)
                                })
                            sum[R](map[I, R](cols2, det_row_sum_term[R, I](det_row_replace_entry[R, I](entry, r, det_row_fn_smul[R, I](c, g)), head, tail, cols2))) =
                                sum[R](map[I, R](cols2, function(col: I) {
                                    c * det_row_sum_term[R, I](det_row_replace_entry[R, I](entry, r, g), head, tail, cols2, col)
                                }))
                            sum_map_scalar_mul[I, R](cols2, c, det_row_sum_term[R, I](det_row_replace_entry[R, I](entry, r, g), head, tail, cols2))
                            c * sum[R](map[I, R](cols2, det_row_sum_term[R, I](det_row_replace_entry[R, I](entry, r, g), head, tail, cols2))) =
                                sum[R](map[I, R](cols2, function(col: I) {
                                    c * det_row_sum_term[R, I](det_row_replace_entry[R, I](entry, r, g), head, tail, cols2, col)
                                }))
                            sum[R](map[I, R](cols2, det_row_sum_term[R, I](det_row_replace_entry[R, I](entry, r, det_row_fn_smul[R, I](c, g)), head, tail, cols2))) =
                                c * sum[R](map[I, R](cols2, det_row_sum_term[R, I](det_row_replace_entry[R, I](entry, r, g), head, tail, cols2)))
                            determinant_list[R, I](det_row_replace_entry[R, I](entry, r, det_row_fn_smul[R, I](c, g)), List.cons(head, tail), cols2) =
                                c * determinant_list[R, I](det_row_replace_entry[R, I](entry, r, g), List.cons(head, tail), cols2)
                        }
                    }
                    forall(cols2: List[I]) {
                        determinant_list[R, I](det_row_replace_entry[R, I](entry, r, det_row_fn_smul[R, I](c, g)), List.cons(head, tail), cols2) =
                            c * determinant_list[R, I](det_row_replace_entry[R, I](entry, r, g), List.cons(head, tail), cols2)
                    }
                }
                p(tail) implies p(List.cons(head, tail))
            }
        }
        p(List.nil[I]) and forall(head: I, tail: List[I]) { p(tail) implies p(List.cons(head, tail)) }
        List.induction(p)
        forall(l: List[I]) { p(l) }
        p(rows)
        forall(cols2: List[I]) {
            determinant_list[R, I](det_row_replace_entry[R, I](entry, r, det_row_fn_smul[R, I](c, g)), rows, cols2) =
                c * determinant_list[R, I](det_row_replace_entry[R, I](entry, r, g), rows, cols2)
        }
        determinant_list[R, I](det_row_replace_entry[R, I](entry, r, det_row_fn_smul[R, I](c, g)), rows, cols) =
            c * determinant_list[R, I](det_row_replace_entry[R, I](entry, r, g), rows, cols)
    }
}

/// The matrix obtained by replacing one row with a supplied row function.
define matrix_row_replace[R: Ring](n: Nat, a: Matrix[R, n.suc, n.suc], r: Fin[n.suc], g: (Fin[n.suc]) -> R) -> Matrix[R, n.suc, n.suc] {
    Matrix[R, n.suc, n.suc].new(det_row_replace_entry[R, Fin[n.suc]](square_matrix_entry_function[R](n.suc, a), r, g))
}

/// The canonical determinant is additive in a single row.
theorem matrix_det_suc_row_add[R: Ring](n: Nat, a: Matrix[R, n.suc, n.suc], r: Fin[n.suc], x: (Fin[n.suc]) -> R, y: (Fin[n.suc]) -> R) {
    matrix_det_suc[R](n, matrix_row_replace[R](n, a, r, det_row_fn_add[R, Fin[n.suc]](x, y))) =
        matrix_det_suc[R](n, matrix_row_replace[R](n, a, r, x)) +
            matrix_det_suc[R](n, matrix_row_replace[R](n, a, r, y))
} by {
    fin_enum_suc_contains(n, r)
    fin_enum_suc(n).contains(r)
    fin_enum_suc_is_unique(n)
    fin_enum_suc(n).is_unique
    remove_one_unique_not_contains_self[Fin[n.suc]](fin_enum_suc(n), r)
    not fin_enum_suc(n).remove_one(r).contains(r)
    determinant_list_row_add[R, Fin[n.suc]](square_matrix_entry_function[R](n.suc, a), x, y, r, fin_enum_suc(n), fin_enum_suc(n))
    determinant_list[R, Fin[n.suc]](det_row_replace_entry[R, Fin[n.suc]](square_matrix_entry_function[R](n.suc, a), r, det_row_fn_add[R, Fin[n.suc]](x, y)), fin_enum_suc(n), fin_enum_suc(n)) =
        determinant_list[R, Fin[n.suc]](det_row_replace_entry[R, Fin[n.suc]](square_matrix_entry_function[R](n.suc, a), r, x), fin_enum_suc(n), fin_enum_suc(n)) +
            determinant_list[R, Fin[n.suc]](det_row_replace_entry[R, Fin[n.suc]](square_matrix_entry_function[R](n.suc, a), r, y), fin_enum_suc(n), fin_enum_suc(n))
    forall(i: Fin[n.suc], c: Fin[n.suc]) {
        square_matrix_entry_function[R](n.suc, matrix_row_replace[R](n, a, r, det_row_fn_add[R, Fin[n.suc]](x, y)), i, c) =
            det_row_replace_entry[R, Fin[n.suc]](square_matrix_entry_function[R](n.suc, a), r, det_row_fn_add[R, Fin[n.suc]](x, y), i, c)
    }
    determinant_list_pointwise[R, Fin[n.suc]](
        square_matrix_entry_function[R](n.suc, matrix_row_replace[R](n, a, r, det_row_fn_add[R, Fin[n.suc]](x, y))),
        det_row_replace_entry[R, Fin[n.suc]](square_matrix_entry_function[R](n.suc, a), r, det_row_fn_add[R, Fin[n.suc]](x, y)),
        fin_enum_suc(n), fin_enum_suc(n))
    determinant_list[R, Fin[n.suc]](square_matrix_entry_function[R](n.suc, matrix_row_replace[R](n, a, r, det_row_fn_add[R, Fin[n.suc]](x, y))), fin_enum_suc(n), fin_enum_suc(n)) =
        determinant_list[R, Fin[n.suc]](det_row_replace_entry[R, Fin[n.suc]](square_matrix_entry_function[R](n.suc, a), r, det_row_fn_add[R, Fin[n.suc]](x, y)), fin_enum_suc(n), fin_enum_suc(n))
    forall(i: Fin[n.suc], c: Fin[n.suc]) {
        square_matrix_entry_function[R](n.suc, matrix_row_replace[R](n, a, r, x), i, c) =
            det_row_replace_entry[R, Fin[n.suc]](square_matrix_entry_function[R](n.suc, a), r, x, i, c)
    }
    determinant_list_pointwise[R, Fin[n.suc]](
        square_matrix_entry_function[R](n.suc, matrix_row_replace[R](n, a, r, x)),
        det_row_replace_entry[R, Fin[n.suc]](square_matrix_entry_function[R](n.suc, a), r, x),
        fin_enum_suc(n), fin_enum_suc(n))
    determinant_list[R, Fin[n.suc]](det_row_replace_entry[R, Fin[n.suc]](square_matrix_entry_function[R](n.suc, a), r, x), fin_enum_suc(n), fin_enum_suc(n)) =
        determinant_list[R, Fin[n.suc]](square_matrix_entry_function[R](n.suc, matrix_row_replace[R](n, a, r, x)), fin_enum_suc(n), fin_enum_suc(n))
    forall(i: Fin[n.suc], c: Fin[n.suc]) {
        square_matrix_entry_function[R](n.suc, matrix_row_replace[R](n, a, r, y), i, c) =
            det_row_replace_entry[R, Fin[n.suc]](square_matrix_entry_function[R](n.suc, a), r, y, i, c)
    }
    determinant_list_pointwise[R, Fin[n.suc]](
        square_matrix_entry_function[R](n.suc, matrix_row_replace[R](n, a, r, y)),
        det_row_replace_entry[R, Fin[n.suc]](square_matrix_entry_function[R](n.suc, a), r, y),
        fin_enum_suc(n), fin_enum_suc(n))
    determinant_list[R, Fin[n.suc]](det_row_replace_entry[R, Fin[n.suc]](square_matrix_entry_function[R](n.suc, a), r, y), fin_enum_suc(n), fin_enum_suc(n)) =
        determinant_list[R, Fin[n.suc]](square_matrix_entry_function[R](n.suc, matrix_row_replace[R](n, a, r, y)), fin_enum_suc(n), fin_enum_suc(n))
    determinant_list[R, Fin[n.suc]](det_row_replace_entry[R, Fin[n.suc]](square_matrix_entry_function[R](n.suc, a), r, det_row_fn_add[R, Fin[n.suc]](x, y)), fin_enum_suc(n), fin_enum_suc(n)) =
        determinant_list[R, Fin[n.suc]](square_matrix_entry_function[R](n.suc, matrix_row_replace[R](n, a, r, x)), fin_enum_suc(n), fin_enum_suc(n)) +
            determinant_list[R, Fin[n.suc]](square_matrix_entry_function[R](n.suc, matrix_row_replace[R](n, a, r, y)), fin_enum_suc(n), fin_enum_suc(n))
    matrix_det_suc_unfold[R](n, matrix_row_replace[R](n, a, r, det_row_fn_add[R, Fin[n.suc]](x, y)))
    matrix_det_suc[R](n, matrix_row_replace[R](n, a, r, det_row_fn_add[R, Fin[n.suc]](x, y))) =
        matrix_det_list[R](n.suc, matrix_row_replace[R](n, a, r, det_row_fn_add[R, Fin[n.suc]](x, y)), fin_enum_suc(n), fin_enum_suc(n))
    matrix_det_list[R](n.suc, matrix_row_replace[R](n, a, r, det_row_fn_add[R, Fin[n.suc]](x, y)), fin_enum_suc(n), fin_enum_suc(n)) =
        determinant_list[R, Fin[n.suc]](square_matrix_entry_function[R](n.suc, matrix_row_replace[R](n, a, r, det_row_fn_add[R, Fin[n.suc]](x, y))), fin_enum_suc(n), fin_enum_suc(n))
    matrix_det_suc[R](n, matrix_row_replace[R](n, a, r, det_row_fn_add[R, Fin[n.suc]](x, y))) =
        determinant_list[R, Fin[n.suc]](square_matrix_entry_function[R](n.suc, matrix_row_replace[R](n, a, r, det_row_fn_add[R, Fin[n.suc]](x, y))), fin_enum_suc(n), fin_enum_suc(n))
    matrix_det_suc_unfold[R](n, matrix_row_replace[R](n, a, r, x))
    matrix_det_suc[R](n, matrix_row_replace[R](n, a, r, x)) =
        matrix_det_list[R](n.suc, matrix_row_replace[R](n, a, r, x), fin_enum_suc(n), fin_enum_suc(n))
    matrix_det_list[R](n.suc, matrix_row_replace[R](n, a, r, x), fin_enum_suc(n), fin_enum_suc(n)) =
        determinant_list[R, Fin[n.suc]](square_matrix_entry_function[R](n.suc, matrix_row_replace[R](n, a, r, x)), fin_enum_suc(n), fin_enum_suc(n))
    matrix_det_suc[R](n, matrix_row_replace[R](n, a, r, x)) =
        determinant_list[R, Fin[n.suc]](square_matrix_entry_function[R](n.suc, matrix_row_replace[R](n, a, r, x)), fin_enum_suc(n), fin_enum_suc(n))
    matrix_det_suc_unfold[R](n, matrix_row_replace[R](n, a, r, y))
    matrix_det_suc[R](n, matrix_row_replace[R](n, a, r, y)) =
        matrix_det_list[R](n.suc, matrix_row_replace[R](n, a, r, y), fin_enum_suc(n), fin_enum_suc(n))
    matrix_det_list[R](n.suc, matrix_row_replace[R](n, a, r, y), fin_enum_suc(n), fin_enum_suc(n)) =
        determinant_list[R, Fin[n.suc]](square_matrix_entry_function[R](n.suc, matrix_row_replace[R](n, a, r, y)), fin_enum_suc(n), fin_enum_suc(n))
    matrix_det_suc[R](n, matrix_row_replace[R](n, a, r, y)) =
        determinant_list[R, Fin[n.suc]](square_matrix_entry_function[R](n.suc, matrix_row_replace[R](n, a, r, y)), fin_enum_suc(n), fin_enum_suc(n))
    matrix_det_suc[R](n, matrix_row_replace[R](n, a, r, det_row_fn_add[R, Fin[n.suc]](x, y))) =
        matrix_det_suc[R](n, matrix_row_replace[R](n, a, r, x)) +
            matrix_det_suc[R](n, matrix_row_replace[R](n, a, r, y))
}

/// The canonical determinant is homogeneous in a single row.
theorem matrix_det_suc_row_smul[R: CommRing](n: Nat, a: Matrix[R, n.suc, n.suc], r: Fin[n.suc], c: R, x: (Fin[n.suc]) -> R) {
    matrix_det_suc[R](n, matrix_row_replace[R](n, a, r, det_row_fn_smul[R, Fin[n.suc]](c, x))) =
        c * matrix_det_suc[R](n, matrix_row_replace[R](n, a, r, x))
} by {
    fin_enum_suc_contains(n, r)
    fin_enum_suc(n).contains(r)
    fin_enum_suc_is_unique(n)
    fin_enum_suc(n).is_unique
    remove_one_unique_not_contains_self[Fin[n.suc]](fin_enum_suc(n), r)
    not fin_enum_suc(n).remove_one(r).contains(r)
    determinant_list_row_smul[R, Fin[n.suc]](square_matrix_entry_function[R](n.suc, a), c, x, r, fin_enum_suc(n), fin_enum_suc(n))
    determinant_list[R, Fin[n.suc]](det_row_replace_entry[R, Fin[n.suc]](square_matrix_entry_function[R](n.suc, a), r, det_row_fn_smul[R, Fin[n.suc]](c, x)), fin_enum_suc(n), fin_enum_suc(n)) =
        c * determinant_list[R, Fin[n.suc]](det_row_replace_entry[R, Fin[n.suc]](square_matrix_entry_function[R](n.suc, a), r, x), fin_enum_suc(n), fin_enum_suc(n))
    forall(i: Fin[n.suc], c2: Fin[n.suc]) {
        square_matrix_entry_function[R](n.suc, matrix_row_replace[R](n, a, r, det_row_fn_smul[R, Fin[n.suc]](c, x)), i, c2) =
            det_row_replace_entry[R, Fin[n.suc]](square_matrix_entry_function[R](n.suc, a), r, det_row_fn_smul[R, Fin[n.suc]](c, x), i, c2)
    }
    determinant_list_pointwise[R, Fin[n.suc]](
        square_matrix_entry_function[R](n.suc, matrix_row_replace[R](n, a, r, det_row_fn_smul[R, Fin[n.suc]](c, x))),
        det_row_replace_entry[R, Fin[n.suc]](square_matrix_entry_function[R](n.suc, a), r, det_row_fn_smul[R, Fin[n.suc]](c, x)),
        fin_enum_suc(n), fin_enum_suc(n))
    determinant_list[R, Fin[n.suc]](square_matrix_entry_function[R](n.suc, matrix_row_replace[R](n, a, r, det_row_fn_smul[R, Fin[n.suc]](c, x))), fin_enum_suc(n), fin_enum_suc(n)) =
        determinant_list[R, Fin[n.suc]](det_row_replace_entry[R, Fin[n.suc]](square_matrix_entry_function[R](n.suc, a), r, det_row_fn_smul[R, Fin[n.suc]](c, x)), fin_enum_suc(n), fin_enum_suc(n))
    forall(i: Fin[n.suc], c2: Fin[n.suc]) {
        square_matrix_entry_function[R](n.suc, matrix_row_replace[R](n, a, r, x), i, c2) =
            det_row_replace_entry[R, Fin[n.suc]](square_matrix_entry_function[R](n.suc, a), r, x, i, c2)
    }
    determinant_list_pointwise[R, Fin[n.suc]](
        square_matrix_entry_function[R](n.suc, matrix_row_replace[R](n, a, r, x)),
        det_row_replace_entry[R, Fin[n.suc]](square_matrix_entry_function[R](n.suc, a), r, x),
        fin_enum_suc(n), fin_enum_suc(n))
    determinant_list[R, Fin[n.suc]](det_row_replace_entry[R, Fin[n.suc]](square_matrix_entry_function[R](n.suc, a), r, x), fin_enum_suc(n), fin_enum_suc(n)) =
        determinant_list[R, Fin[n.suc]](square_matrix_entry_function[R](n.suc, matrix_row_replace[R](n, a, r, x)), fin_enum_suc(n), fin_enum_suc(n))
    determinant_list[R, Fin[n.suc]](det_row_replace_entry[R, Fin[n.suc]](square_matrix_entry_function[R](n.suc, a), r, det_row_fn_smul[R, Fin[n.suc]](c, x)), fin_enum_suc(n), fin_enum_suc(n)) =
        c * determinant_list[R, Fin[n.suc]](square_matrix_entry_function[R](n.suc, matrix_row_replace[R](n, a, r, x)), fin_enum_suc(n), fin_enum_suc(n))
    matrix_det_suc_unfold[R](n, matrix_row_replace[R](n, a, r, det_row_fn_smul[R, Fin[n.suc]](c, x)))
    matrix_det_suc[R](n, matrix_row_replace[R](n, a, r, det_row_fn_smul[R, Fin[n.suc]](c, x))) =
        matrix_det_list[R](n.suc, matrix_row_replace[R](n, a, r, det_row_fn_smul[R, Fin[n.suc]](c, x)), fin_enum_suc(n), fin_enum_suc(n))
    matrix_det_list[R](n.suc, matrix_row_replace[R](n, a, r, det_row_fn_smul[R, Fin[n.suc]](c, x)), fin_enum_suc(n), fin_enum_suc(n)) =
        determinant_list[R, Fin[n.suc]](square_matrix_entry_function[R](n.suc, matrix_row_replace[R](n, a, r, det_row_fn_smul[R, Fin[n.suc]](c, x))), fin_enum_suc(n), fin_enum_suc(n))
    matrix_det_suc[R](n, matrix_row_replace[R](n, a, r, det_row_fn_smul[R, Fin[n.suc]](c, x))) =
        determinant_list[R, Fin[n.suc]](square_matrix_entry_function[R](n.suc, matrix_row_replace[R](n, a, r, det_row_fn_smul[R, Fin[n.suc]](c, x))), fin_enum_suc(n), fin_enum_suc(n))
    matrix_det_suc_unfold[R](n, matrix_row_replace[R](n, a, r, x))
    matrix_det_suc[R](n, matrix_row_replace[R](n, a, r, x)) =
        matrix_det_list[R](n.suc, matrix_row_replace[R](n, a, r, x), fin_enum_suc(n), fin_enum_suc(n))
    matrix_det_list[R](n.suc, matrix_row_replace[R](n, a, r, x), fin_enum_suc(n), fin_enum_suc(n)) =
        determinant_list[R, Fin[n.suc]](square_matrix_entry_function[R](n.suc, matrix_row_replace[R](n, a, r, x)), fin_enum_suc(n), fin_enum_suc(n))
    matrix_det_suc[R](n, matrix_row_replace[R](n, a, r, x)) =
        determinant_list[R, Fin[n.suc]](square_matrix_entry_function[R](n.suc, matrix_row_replace[R](n, a, r, x)), fin_enum_suc(n), fin_enum_suc(n))
    matrix_det_suc[R](n, matrix_row_replace[R](n, a, r, det_row_fn_smul[R, Fin[n.suc]](c, x))) =
        c * matrix_det_suc[R](n, matrix_row_replace[R](n, a, r, x))
}

// ---------------------------------------------------------------------------
// Status note (do not delete): multiplicative property of the determinant.
//
// The row-multilinearity lemmas above (determinant_list_row_add,
// determinant_list_row_smul, matrix_det_suc_row_add, matrix_det_suc_row_smul)
// are the foundation for proving det(A.B) = det(A).det(B) by the
// characterization route:
//
//   Let F_A(B) = matrix_det_suc(n, A.mul(B)).  If we know that any function
//   on (n+1)x(n+1) matrices that is (i) multilinear in every row, (ii)
//   alternating (equal rows give zero, swapping two rows negates), and (iii)
//   sends the identity to 1, is equal to matrix_det_suc, then F_A(B) =
//   matrix_det_suc(B).F_A(1) and multiplicativity follows.
//
// The missing pieces, in order of dependency:
//   1. The Leibniz formula for the list-based determinant:
//        determinant_list(e, rows, cols) = sum over permutations sigma of
//        sign(sigma) . product_i e(rows_i, cols_{sigma(i)}),
//      restricted to the canonical enumerations.  This requires the sign of a
//      permutation of Fin[n] (well-definedness, sign of a transposition is
//      -1, sign(sigma o tau) = sign(sigma).sign(tau)).  There is currently NO
//      permutation-sign infrastructure anywhere in the library (only
//      List.is_permutation, which is count-based multiset equality, and
//      alternating_sign for (-1)^k).  The existing row-swap lemmas cover only
//      the first two rows (determinant_list_swap_first_two_rows), and
//      generalising them needs exactly this theory.
//   2. General equal-rows: two equal rows give determinant zero.  The
//      natural proof pairs sigma with sigma o (ij) in the Leibniz sum; even
//      for equal rows in the first two positions the naive double-sum
//      cancellation only yields 2.det = 0, which is not zero in general
//      characteristic, so the unordered-pair formulation (or Leibniz) is
//      required.
//   3. The characterization theorem and the application to F_A.
//
// All three routes to det(A.B) = det(A).det(B) (characterization via
// multilinearity, direct Cauchy-Binet expansion, or Leibniz expansion of the
// product) pass through the Leibniz formula, so the sign-of-permutation
// theory is the true bottleneck, not the list-based definition of the
// determinant itself.
// ---------------------------------------------------------------------------

// ---------------------------------------------------------------------------
// Matrix multiplication associativity
// ---------------------------------------------------------------------------
//
// The inverse properties need `(A*B)*C = A*(B*C)`.  The proof is the
// double-sum rearrangement `sum_k (sum_l a(i,l) b(l,k)) c(k,j) = sum_l a(i,l)
// (sum_k b(l,k) c(k,j))`, carried out on the list form of the finite sums.

/// A sum factors a right scalar out of a mapped list sum.
theorem sum_map_scalar_mul_right[T, R: Semiring](items: List[T], c: R, f: T -> R) {
    sum[R](map[T, R](items, f)) * c = sum[R](map[T, R](items, function(x: T) { f(x) * c }))
} by {
    define p(l: List[T]) -> Bool {
        sum[R](map[T, R](l, f)) * c = sum[R](map[T, R](l, function(x: T) { f(x) * c }))
    }
    map[T, R](List.nil[T], f) = List.nil[R]
    map[T, R](List.nil[T], function(x: T) { f(x) * c }) = List.nil[R]
    sum[R](List.nil[R]) = R.0
    R.0 * c = R.0
    p(List.nil[T])
    forall(head: T, tail: List[T]) {
        if p(tail) {
            p(tail) = (sum[R](map[T, R](tail, f)) * c = sum[R](map[T, R](tail, function(x: T) { f(x) * c })))
            sum_map_remove_one[T, R](List.cons(head, tail), head, f)
            List.cons(head, tail).contains(head)
            f(head) + sum[R](map[T, R](List.cons(head, tail).remove_one(head), f)) =
                sum[R](map[T, R](List.cons(head, tail), f))
            remove_one_cons_eq(head, tail)
            List.cons(head, tail).remove_one(head) = tail
            f(head) + sum[R](map[T, R](tail, f)) = sum[R](map[T, R](List.cons(head, tail), f))
            (f(head) + sum[R](map[T, R](tail, f))) * c = f(head) * c + sum[R](map[T, R](tail, f)) * c
            sum[R](map[T, R](List.cons(head, tail), f)) * c = f(head) * c + sum[R](map[T, R](tail, f)) * c
            sum[R](map[T, R](tail, f)) * c = sum[R](map[T, R](tail, function(x: T) { f(x) * c }))
            f(head) * c + sum[R](map[T, R](tail, f)) * c = f(head) * c + sum[R](map[T, R](tail, function(x: T) { f(x) * c }))
            sum[R](map[T, R](List.cons(head, tail), f)) * c = f(head) * c + sum[R](map[T, R](tail, function(x: T) { f(x) * c }))
            sum_map_remove_one[T, R](List.cons(head, tail), head, function(x: T) { f(x) * c })
            f(head) * c + sum[R](map[T, R](List.cons(head, tail).remove_one(head), function(x: T) { f(x) * c })) =
                sum[R](map[T, R](List.cons(head, tail), function(x: T) { f(x) * c }))
            f(head) * c + sum[R](map[T, R](tail, function(x: T) { f(x) * c })) =
                sum[R](map[T, R](List.cons(head, tail), function(x: T) { f(x) * c }))
            sum[R](map[T, R](List.cons(head, tail), f)) * c =
                sum[R](map[T, R](List.cons(head, tail), function(x: T) { f(x) * c }))
            p(List.cons(head, tail))
        }
        p(tail) implies p(List.cons(head, tail))
    }
    p(List.nil[T]) and forall(head: T, tail: List[T]) { p(tail) implies p(List.cons(head, tail)) }
    List.induction(p)
    forall(l: List[T]) { p(l) }
    p(items)
    sum[R](map[T, R](items, f)) * c = sum[R](map[T, R](items, function(x: T) { f(x) * c }))
}

/// A sum factors a left scalar out of a mapped list sum.
theorem sum_map_scalar_mul_semiring_left[T, R: Semiring](items: List[T], c: R, f: T -> R) {
    c * sum[R](map[T, R](items, f)) = sum[R](map[T, R](items, function(x: T) { c * f(x) }))
} by {
    define p(l: List[T]) -> Bool {
        c * sum[R](map[T, R](l, f)) = sum[R](map[T, R](l, function(x: T) { c * f(x) }))
    }
    map[T, R](List.nil[T], f) = List.nil[R]
    map[T, R](List.nil[T], function(x: T) { c * f(x) }) = List.nil[R]
    sum[R](List.nil[R]) = R.0
    c * R.0 = R.0
    p(List.nil[T])
    forall(head: T, tail: List[T]) {
        if p(tail) {
            p(tail) = (c * sum[R](map[T, R](tail, f)) = sum[R](map[T, R](tail, function(x: T) { c * f(x) })))
            sum_map_remove_one[T, R](List.cons(head, tail), head, f)
            List.cons(head, tail).contains(head)
            f(head) + sum[R](map[T, R](List.cons(head, tail).remove_one(head), f)) =
                sum[R](map[T, R](List.cons(head, tail), f))
            remove_one_cons_eq(head, tail)
            List.cons(head, tail).remove_one(head) = tail
            f(head) + sum[R](map[T, R](tail, f)) = sum[R](map[T, R](List.cons(head, tail), f))
            c * (f(head) + sum[R](map[T, R](tail, f))) = c * f(head) + c * sum[R](map[T, R](tail, f))
            c * sum[R](map[T, R](List.cons(head, tail), f)) = c * f(head) + c * sum[R](map[T, R](tail, f))
            c * sum[R](map[T, R](tail, f)) = sum[R](map[T, R](tail, function(x: T) { c * f(x) }))
            c * f(head) + c * sum[R](map[T, R](tail, f)) = c * f(head) + sum[R](map[T, R](tail, function(x: T) { c * f(x) }))
            c * sum[R](map[T, R](List.cons(head, tail), f)) = c * f(head) + sum[R](map[T, R](tail, function(x: T) { c * f(x) }))
            sum_map_remove_one[T, R](List.cons(head, tail), head, function(x: T) { c * f(x) })
            c * f(head) + sum[R](map[T, R](List.cons(head, tail).remove_one(head), function(x: T) { c * f(x) })) =
                sum[R](map[T, R](List.cons(head, tail), function(x: T) { c * f(x) }))
            c * f(head) + sum[R](map[T, R](tail, function(x: T) { c * f(x) })) =
                sum[R](map[T, R](List.cons(head, tail), function(x: T) { c * f(x) }))
            c * sum[R](map[T, R](List.cons(head, tail), f)) =
                sum[R](map[T, R](List.cons(head, tail), function(x: T) { c * f(x) }))
            p(List.cons(head, tail))
        }
        p(tail) implies p(List.cons(head, tail))
    }
    p(List.nil[T]) and forall(head: T, tail: List[T]) { p(tail) implies p(List.cons(head, tail)) }
    List.induction(p)
    forall(l: List[T]) { p(l) }
    p(items)
    c * sum[R](map[T, R](items, f)) = sum[R](map[T, R](items, function(x: T) { c * f(x) }))
}

/// The entries of `(A*B)*C` and `A*(B*C)` agree pointwise.
theorem matrix_mul_square_assoc_entry[R: Semiring](
    n: Nat, a: Matrix[R, n.suc, n.suc], b: Matrix[R, n.suc, n.suc],
    c: Matrix[R, n.suc, n.suc], i: Fin[n.suc], j: Fin[n.suc]
) {
    matrix_entry[R](n.suc, n.suc, matrix_mul[R](n.suc, n.suc, n.suc,
        matrix_mul[R](n.suc, n.suc, n.suc, a, b), c), i, j) =
    matrix_entry[R](n.suc, n.suc, matrix_mul[R](n.suc, n.suc, n.suc, a,
        matrix_mul[R](n.suc, n.suc, n.suc, b, c)), i, j)
} by {
    let ab = matrix_mul[R](n.suc, n.suc, n.suc, a, b)
    let bc = matrix_mul[R](n.suc, n.suc, n.suc, b, c)
    let lhs = matrix_mul[R](n.suc, n.suc, n.suc, ab, c)
    let rhs = matrix_mul[R](n.suc, n.suc, n.suc, a, bc)
    matrix_entry_mul[R](n.suc, n.suc, n.suc, ab, c, i, j)
    matrix_entry[R](n.suc, n.suc, lhs, i, j) = fin_sum[R](n.suc, matrix_mul_term[R](n.suc, n.suc, n.suc, ab, c, i, j))
    fin_sum_suc_eq_sum_fin_enum_suc[R](n, matrix_mul_term[R](n.suc, n.suc, n.suc, ab, c, i, j))
    fin_sum[R](n.suc, matrix_mul_term[R](n.suc, n.suc, n.suc, ab, c, i, j)) =
        sum[R](map[Fin[n.suc], R](fin_enum_suc(n), matrix_mul_term[R](n.suc, n.suc, n.suc, ab, c, i, j)))
    matrix_entry[R](n.suc, n.suc, lhs, i, j) =
        sum[R](map[Fin[n.suc], R](fin_enum_suc(n), matrix_mul_term[R](n.suc, n.suc, n.suc, ab, c, i, j)))
    // each outer term (sum_l a(i,l) b(l,k)) * c(k,j) becomes sum_l a(i,l) b(l,k) * c(k,j)
    forall(k: Fin[n.suc]) {
        matrix_mul_term[R](n.suc, n.suc, n.suc, ab, c, i, j, k) =
            matrix_entry[R](n.suc, n.suc, ab, i, k) * matrix_entry[R](n.suc, n.suc, c, k, j)
        matrix_entry_mul[R](n.suc, n.suc, n.suc, a, b, i, k)
        matrix_entry[R](n.suc, n.suc, ab, i, k) = fin_sum[R](n.suc, matrix_mul_term[R](n.suc, n.suc, n.suc, a, b, i, k))
        fin_sum_suc_eq_sum_fin_enum_suc[R](n, matrix_mul_term[R](n.suc, n.suc, n.suc, a, b, i, k))
        fin_sum[R](n.suc, matrix_mul_term[R](n.suc, n.suc, n.suc, a, b, i, k)) =
            sum[R](map[Fin[n.suc], R](fin_enum_suc(n), matrix_mul_term[R](n.suc, n.suc, n.suc, a, b, i, k)))
        matrix_entry[R](n.suc, n.suc, ab, i, k) =
            sum[R](map[Fin[n.suc], R](fin_enum_suc(n), matrix_mul_term[R](n.suc, n.suc, n.suc, a, b, i, k)))
        sum_map_scalar_mul_right[Fin[n.suc], R](fin_enum_suc(n), matrix_entry[R](n.suc, n.suc, c, k, j),
            matrix_mul_term[R](n.suc, n.suc, n.suc, a, b, i, k))
        sum[R](map[Fin[n.suc], R](fin_enum_suc(n), matrix_mul_term[R](n.suc, n.suc, n.suc, a, b, i, k))) *
            matrix_entry[R](n.suc, n.suc, c, k, j) =
            sum[R](map[Fin[n.suc], R](fin_enum_suc(n), function(x: Fin[n.suc]) {
                matrix_mul_term[R](n.suc, n.suc, n.suc, a, b, i, k, x) * matrix_entry[R](n.suc, n.suc, c, k, j)
            }))
        matrix_entry[R](n.suc, n.suc, ab, i, k) * matrix_entry[R](n.suc, n.suc, c, k, j) =
            sum[R](map[Fin[n.suc], R](fin_enum_suc(n), function(x: Fin[n.suc]) {
                matrix_mul_term[R](n.suc, n.suc, n.suc, a, b, i, k, x) * matrix_entry[R](n.suc, n.suc, c, k, j)
            }))
        matrix_mul_term[R](n.suc, n.suc, n.suc, ab, c, i, j, k) =
            sum[R](map[Fin[n.suc], R](fin_enum_suc(n), function(x: Fin[n.suc]) {
                matrix_mul_term[R](n.suc, n.suc, n.suc, a, b, i, k, x) * matrix_entry[R](n.suc, n.suc, c, k, j)
            }))
        forall(x: Fin[n.suc]) {
            matrix_mul_term[R](n.suc, n.suc, n.suc, a, b, i, k, x) =
                matrix_entry[R](n.suc, n.suc, a, i, x) * matrix_entry[R](n.suc, n.suc, b, x, k)
            matrix_mul_term[R](n.suc, n.suc, n.suc, a, b, i, k, x) * matrix_entry[R](n.suc, n.suc, c, k, j) =
                (matrix_entry[R](n.suc, n.suc, a, i, x) * matrix_entry[R](n.suc, n.suc, b, x, k)) *
                    matrix_entry[R](n.suc, n.suc, c, k, j)
            (matrix_entry[R](n.suc, n.suc, a, i, x) * matrix_entry[R](n.suc, n.suc, b, x, k)) *
                matrix_entry[R](n.suc, n.suc, c, k, j) =
                matrix_entry[R](n.suc, n.suc, a, i, x) *
                    (matrix_entry[R](n.suc, n.suc, b, x, k) * matrix_entry[R](n.suc, n.suc, c, k, j))
            matrix_mul_term[R](n.suc, n.suc, n.suc, a, b, i, k, x) * matrix_entry[R](n.suc, n.suc, c, k, j) =
                matrix_entry[R](n.suc, n.suc, a, i, x) *
                    (matrix_entry[R](n.suc, n.suc, b, x, k) * matrix_entry[R](n.suc, n.suc, c, k, j))
        }
        sum_map_of_pointwise[Fin[n.suc], R](fin_enum_suc(n),
            function(x: Fin[n.suc]) {
                matrix_mul_term[R](n.suc, n.suc, n.suc, a, b, i, k, x) * matrix_entry[R](n.suc, n.suc, c, k, j)
            },
            function(x: Fin[n.suc]) {
                matrix_entry[R](n.suc, n.suc, a, i, x) *
                    (matrix_entry[R](n.suc, n.suc, b, x, k) * matrix_entry[R](n.suc, n.suc, c, k, j))
            })
        sum[R](map[Fin[n.suc], R](fin_enum_suc(n), function(x: Fin[n.suc]) {
            matrix_mul_term[R](n.suc, n.suc, n.suc, a, b, i, k, x) * matrix_entry[R](n.suc, n.suc, c, k, j)
        })) = sum[R](map[Fin[n.suc], R](fin_enum_suc(n), function(x: Fin[n.suc]) {
            matrix_entry[R](n.suc, n.suc, a, i, x) *
                (matrix_entry[R](n.suc, n.suc, b, x, k) * matrix_entry[R](n.suc, n.suc, c, k, j))
        }))
        matrix_mul_term[R](n.suc, n.suc, n.suc, ab, c, i, j, k) =
            sum[R](map[Fin[n.suc], R](fin_enum_suc(n), function(x: Fin[n.suc]) {
                matrix_entry[R](n.suc, n.suc, a, i, x) *
                    (matrix_entry[R](n.suc, n.suc, b, x, k) * matrix_entry[R](n.suc, n.suc, c, k, j))
            }))
    }
    sum_map_of_pointwise[Fin[n.suc], R](fin_enum_suc(n),
        matrix_mul_term[R](n.suc, n.suc, n.suc, ab, c, i, j),
        function(k: Fin[n.suc]) {
            sum[R](map[Fin[n.suc], R](fin_enum_suc(n), function(x: Fin[n.suc]) {
                matrix_entry[R](n.suc, n.suc, a, i, x) *
                    (matrix_entry[R](n.suc, n.suc, b, x, k) * matrix_entry[R](n.suc, n.suc, c, k, j))
            }))
        })
    sum[R](map[Fin[n.suc], R](fin_enum_suc(n), matrix_mul_term[R](n.suc, n.suc, n.suc, ab, c, i, j))) =
        sum[R](map[Fin[n.suc], R](fin_enum_suc(n), function(k: Fin[n.suc]) {
            sum[R](map[Fin[n.suc], R](fin_enum_suc(n), function(x: Fin[n.suc]) {
                matrix_entry[R](n.suc, n.suc, a, i, x) *
                    (matrix_entry[R](n.suc, n.suc, b, x, k) * matrix_entry[R](n.suc, n.suc, c, k, j))
            }))
        }))
    matrix_entry[R](n.suc, n.suc, lhs, i, j) =
        sum[R](map[Fin[n.suc], R](fin_enum_suc(n), function(k: Fin[n.suc]) {
            sum[R](map[Fin[n.suc], R](fin_enum_suc(n), function(x: Fin[n.suc]) {
                matrix_entry[R](n.suc, n.suc, a, i, x) *
                    (matrix_entry[R](n.suc, n.suc, b, x, k) * matrix_entry[R](n.suc, n.suc, c, k, j))
            }))
        }))
    // RHS: a(i,l) * (sum_k b(l,k) c(k,j))
    matrix_entry_mul[R](n.suc, n.suc, n.suc, b, c, i, j)
    matrix_entry[R](n.suc, n.suc, bc, i, j) = fin_sum[R](n.suc, matrix_mul_term[R](n.suc, n.suc, n.suc, b, c, i, j))
    matrix_entry_mul[R](n.suc, n.suc, n.suc, a, bc, i, j)
    matrix_entry[R](n.suc, n.suc, rhs, i, j) = fin_sum[R](n.suc, matrix_mul_term[R](n.suc, n.suc, n.suc, a, bc, i, j))
    fin_sum_suc_eq_sum_fin_enum_suc[R](n, matrix_mul_term[R](n.suc, n.suc, n.suc, a, bc, i, j))
    fin_sum[R](n.suc, matrix_mul_term[R](n.suc, n.suc, n.suc, a, bc, i, j)) =
        sum[R](map[Fin[n.suc], R](fin_enum_suc(n), matrix_mul_term[R](n.suc, n.suc, n.suc, a, bc, i, j)))
    matrix_entry[R](n.suc, n.suc, rhs, i, j) =
        sum[R](map[Fin[n.suc], R](fin_enum_suc(n), matrix_mul_term[R](n.suc, n.suc, n.suc, a, bc, i, j)))
    forall(l: Fin[n.suc]) {
        matrix_mul_term[R](n.suc, n.suc, n.suc, a, bc, i, j, l) =
            matrix_entry[R](n.suc, n.suc, a, i, l) * matrix_entry[R](n.suc, n.suc, bc, l, j)
        matrix_entry_mul[R](n.suc, n.suc, n.suc, b, c, l, j)
        matrix_entry[R](n.suc, n.suc, bc, l, j) = fin_sum[R](n.suc, matrix_mul_term[R](n.suc, n.suc, n.suc, b, c, l, j))
        fin_sum_suc_eq_sum_fin_enum_suc[R](n, matrix_mul_term[R](n.suc, n.suc, n.suc, b, c, l, j))
        fin_sum[R](n.suc, matrix_mul_term[R](n.suc, n.suc, n.suc, b, c, l, j)) =
            sum[R](map[Fin[n.suc], R](fin_enum_suc(n), matrix_mul_term[R](n.suc, n.suc, n.suc, b, c, l, j)))
        matrix_entry[R](n.suc, n.suc, bc, l, j) =
            sum[R](map[Fin[n.suc], R](fin_enum_suc(n), matrix_mul_term[R](n.suc, n.suc, n.suc, b, c, l, j)))
        sum_map_scalar_mul_semiring_left[Fin[n.suc], R](fin_enum_suc(n), matrix_entry[R](n.suc, n.suc, a, i, l),
            function(x: Fin[n.suc]) {
                matrix_mul_term[R](n.suc, n.suc, n.suc, b, c, l, j, x)
            })
        matrix_entry[R](n.suc, n.suc, a, i, l) *
            sum[R](map[Fin[n.suc], R](fin_enum_suc(n), matrix_mul_term[R](n.suc, n.suc, n.suc, b, c, l, j))) =
            sum[R](map[Fin[n.suc], R](fin_enum_suc(n), function(x: Fin[n.suc]) {
                matrix_entry[R](n.suc, n.suc, a, i, l) *
                    matrix_mul_term[R](n.suc, n.suc, n.suc, b, c, l, j, x)
            }))
        matrix_entry[R](n.suc, n.suc, a, i, l) * matrix_entry[R](n.suc, n.suc, bc, l, j) =
            sum[R](map[Fin[n.suc], R](fin_enum_suc(n), function(x: Fin[n.suc]) {
                matrix_entry[R](n.suc, n.suc, a, i, l) *
                    matrix_mul_term[R](n.suc, n.suc, n.suc, b, c, l, j, x)
            }))
        matrix_mul_term[R](n.suc, n.suc, n.suc, a, bc, i, j, l) =
            sum[R](map[Fin[n.suc], R](fin_enum_suc(n), function(x: Fin[n.suc]) {
                matrix_entry[R](n.suc, n.suc, a, i, l) *
                    matrix_mul_term[R](n.suc, n.suc, n.suc, b, c, l, j, x)
            }))
        forall(x: Fin[n.suc]) {
            matrix_mul_term[R](n.suc, n.suc, n.suc, b, c, l, j, x) =
                matrix_entry[R](n.suc, n.suc, b, l, x) * matrix_entry[R](n.suc, n.suc, c, x, j)
            matrix_entry[R](n.suc, n.suc, a, i, l) * matrix_mul_term[R](n.suc, n.suc, n.suc, b, c, l, j, x) =
                matrix_entry[R](n.suc, n.suc, a, i, l) *
                    (matrix_entry[R](n.suc, n.suc, b, l, x) * matrix_entry[R](n.suc, n.suc, c, x, j))
            matrix_entry[R](n.suc, n.suc, a, i, l) *
                (matrix_entry[R](n.suc, n.suc, b, l, x) * matrix_entry[R](n.suc, n.suc, c, x, j)) =
                (matrix_entry[R](n.suc, n.suc, a, i, l) * matrix_entry[R](n.suc, n.suc, b, l, x)) *
                    matrix_entry[R](n.suc, n.suc, c, x, j)
        }
        sum_map_of_pointwise[Fin[n.suc], R](fin_enum_suc(n),
            function(x: Fin[n.suc]) {
                matrix_entry[R](n.suc, n.suc, a, i, l) * matrix_mul_term[R](n.suc, n.suc, n.suc, b, c, l, j, x)
            },
            function(x: Fin[n.suc]) {
                matrix_entry[R](n.suc, n.suc, a, i, l) *
                    (matrix_entry[R](n.suc, n.suc, b, l, x) * matrix_entry[R](n.suc, n.suc, c, x, j))
            })
        sum[R](map[Fin[n.suc], R](fin_enum_suc(n), function(x: Fin[n.suc]) {
            matrix_entry[R](n.suc, n.suc, a, i, l) * matrix_mul_term[R](n.suc, n.suc, n.suc, b, c, l, j, x)
        })) = sum[R](map[Fin[n.suc], R](fin_enum_suc(n), function(x: Fin[n.suc]) {
            matrix_entry[R](n.suc, n.suc, a, i, l) *
                (matrix_entry[R](n.suc, n.suc, b, l, x) * matrix_entry[R](n.suc, n.suc, c, x, j))
        }))
        matrix_mul_term[R](n.suc, n.suc, n.suc, a, bc, i, j, l) =
            sum[R](map[Fin[n.suc], R](fin_enum_suc(n), function(x: Fin[n.suc]) {
                matrix_entry[R](n.suc, n.suc, a, i, l) *
                    (matrix_entry[R](n.suc, n.suc, b, l, x) * matrix_entry[R](n.suc, n.suc, c, x, j))
            }))
    }
    sum_map_of_pointwise[Fin[n.suc], R](fin_enum_suc(n),
        matrix_mul_term[R](n.suc, n.suc, n.suc, a, bc, i, j),
        function(l: Fin[n.suc]) {
            sum[R](map[Fin[n.suc], R](fin_enum_suc(n), function(x: Fin[n.suc]) {
                matrix_entry[R](n.suc, n.suc, a, i, l) *
                    (matrix_entry[R](n.suc, n.suc, b, l, x) * matrix_entry[R](n.suc, n.suc, c, x, j))
            }))
        })
    sum[R](map[Fin[n.suc], R](fin_enum_suc(n), matrix_mul_term[R](n.suc, n.suc, n.suc, a, bc, i, j))) =
        sum[R](map[Fin[n.suc], R](fin_enum_suc(n), function(l: Fin[n.suc]) {
            sum[R](map[Fin[n.suc], R](fin_enum_suc(n), function(x: Fin[n.suc]) {
                matrix_entry[R](n.suc, n.suc, a, i, l) *
                    (matrix_entry[R](n.suc, n.suc, b, l, x) * matrix_entry[R](n.suc, n.suc, c, x, j))
            }))
        }))
    // reindex the LHS double sum, sum_k sum_x -> sum_x sum_k
    sum_map_sum_exchange[Fin[n.suc], R](fin_enum_suc(n), fin_enum_suc(n),
        function(k: Fin[n.suc], x: Fin[n.suc]) {
            matrix_entry[R](n.suc, n.suc, a, i, x) *
                (matrix_entry[R](n.suc, n.suc, b, x, k) * matrix_entry[R](n.suc, n.suc, c, k, j))
        })
    sum[R](map[Fin[n.suc], R](fin_enum_suc(n), function(k: Fin[n.suc]) {
        sum[R](map[Fin[n.suc], R](fin_enum_suc(n), function(x: Fin[n.suc]) {
            matrix_entry[R](n.suc, n.suc, a, i, x) *
                (matrix_entry[R](n.suc, n.suc, b, x, k) * matrix_entry[R](n.suc, n.suc, c, k, j))
        }))
    })) = sum[R](map[Fin[n.suc], R](fin_enum_suc(n), function(x: Fin[n.suc]) {
        sum[R](map[Fin[n.suc], R](fin_enum_suc(n), function(k: Fin[n.suc]) {
            matrix_entry[R](n.suc, n.suc, a, i, x) *
                (matrix_entry[R](n.suc, n.suc, b, x, k) * matrix_entry[R](n.suc, n.suc, c, k, j))
        }))
    }))
    matrix_entry[R](n.suc, n.suc, lhs, i, j) =
        sum[R](map[Fin[n.suc], R](fin_enum_suc(n), function(x: Fin[n.suc]) {
            sum[R](map[Fin[n.suc], R](fin_enum_suc(n), function(k: Fin[n.suc]) {
                matrix_entry[R](n.suc, n.suc, a, i, x) *
                    (matrix_entry[R](n.suc, n.suc, b, x, k) * matrix_entry[R](n.suc, n.suc, c, k, j))
            }))
        }))
    // The two outer sums agree pointwise: the bound variables are renamed.
    forall(v: Fin[n.suc]) {
        sum[R](map[Fin[n.suc], R](fin_enum_suc(n), function(k: Fin[n.suc]) {
            matrix_entry[R](n.suc, n.suc, a, i, v) *
                (matrix_entry[R](n.suc, n.suc, b, v, k) * matrix_entry[R](n.suc, n.suc, c, k, j))
        })) = sum[R](map[Fin[n.suc], R](fin_enum_suc(n), function(x: Fin[n.suc]) {
            matrix_entry[R](n.suc, n.suc, a, i, v) *
                (matrix_entry[R](n.suc, n.suc, b, v, x) * matrix_entry[R](n.suc, n.suc, c, x, j))
        }))
    }
    sum_map_of_pointwise[Fin[n.suc], R](fin_enum_suc(n),
        function(x: Fin[n.suc]) {
            sum[R](map[Fin[n.suc], R](fin_enum_suc(n), function(k: Fin[n.suc]) {
                matrix_entry[R](n.suc, n.suc, a, i, x) *
                    (matrix_entry[R](n.suc, n.suc, b, x, k) * matrix_entry[R](n.suc, n.suc, c, k, j))
            }))
        },
        function(l: Fin[n.suc]) {
            sum[R](map[Fin[n.suc], R](fin_enum_suc(n), function(x: Fin[n.suc]) {
                matrix_entry[R](n.suc, n.suc, a, i, l) *
                    (matrix_entry[R](n.suc, n.suc, b, l, x) * matrix_entry[R](n.suc, n.suc, c, x, j))
            }))
        })
    sum[R](map[Fin[n.suc], R](fin_enum_suc(n), function(x: Fin[n.suc]) {
        sum[R](map[Fin[n.suc], R](fin_enum_suc(n), function(k: Fin[n.suc]) {
            matrix_entry[R](n.suc, n.suc, a, i, x) *
                (matrix_entry[R](n.suc, n.suc, b, x, k) * matrix_entry[R](n.suc, n.suc, c, k, j))
        }))
    })) = sum[R](map[Fin[n.suc], R](fin_enum_suc(n), function(l: Fin[n.suc]) {
        sum[R](map[Fin[n.suc], R](fin_enum_suc(n), function(x: Fin[n.suc]) {
            matrix_entry[R](n.suc, n.suc, a, i, l) *
                (matrix_entry[R](n.suc, n.suc, b, l, x) * matrix_entry[R](n.suc, n.suc, c, x, j))
        }))
    }))
    matrix_entry[R](n.suc, n.suc, lhs, i, j) =
        sum[R](map[Fin[n.suc], R](fin_enum_suc(n), function(l: Fin[n.suc]) {
            sum[R](map[Fin[n.suc], R](fin_enum_suc(n), function(x: Fin[n.suc]) {
                matrix_entry[R](n.suc, n.suc, a, i, l) *
                    (matrix_entry[R](n.suc, n.suc, b, l, x) * matrix_entry[R](n.suc, n.suc, c, x, j))
            }))
        }))
    matrix_entry[R](n.suc, n.suc, rhs, i, j) =
        sum[R](map[Fin[n.suc], R](fin_enum_suc(n), function(l: Fin[n.suc]) {
            sum[R](map[Fin[n.suc], R](fin_enum_suc(n), function(x: Fin[n.suc]) {
                matrix_entry[R](n.suc, n.suc, a, i, l) *
                    (matrix_entry[R](n.suc, n.suc, b, l, x) * matrix_entry[R](n.suc, n.suc, c, x, j))
            }))
        }))
    matrix_entry[R](n.suc, n.suc, lhs, i, j) = matrix_entry[R](n.suc, n.suc, rhs, i, j)
}

/// Matrix multiplication is associative for square matrices.
theorem matrix_mul_square_assoc[R: Semiring](
    n: Nat, a: Matrix[R, n.suc, n.suc], b: Matrix[R, n.suc, n.suc], c: Matrix[R, n.suc, n.suc]
) {
    matrix_mul[R](n.suc, n.suc, n.suc, matrix_mul[R](n.suc, n.suc, n.suc, a, b), c) =
        matrix_mul[R](n.suc, n.suc, n.suc, a, matrix_mul[R](n.suc, n.suc, n.suc, b, c))
} by {
    forall(i: Fin[n.suc], j: Fin[n.suc]) {
        matrix_mul_square_assoc_entry[R](n, a, b, c, i, j)
        matrix_entry[R](n.suc, n.suc, matrix_mul[R](n.suc, n.suc, n.suc,
            matrix_mul[R](n.suc, n.suc, n.suc, a, b), c), i, j) =
            matrix_entry[R](n.suc, n.suc, matrix_mul[R](n.suc, n.suc, n.suc, a,
                matrix_mul[R](n.suc, n.suc, n.suc, b, c)), i, j)
    }
    matrix_ext[R](n.suc, n.suc, matrix_mul[R](n.suc, n.suc, n.suc,
        matrix_mul[R](n.suc, n.suc, n.suc, a, b), c),
        matrix_mul[R](n.suc, n.suc, n.suc, a, matrix_mul[R](n.suc, n.suc, n.suc, b, c)))
    matrix_mul[R](n.suc, n.suc, n.suc, matrix_mul[R](n.suc, n.suc, n.suc, a, b), c) =
        matrix_mul[R](n.suc, n.suc, n.suc, a, matrix_mul[R](n.suc, n.suc, n.suc, b, c))
}

// ---------------------------------------------------------------------------
// Alternation: a determinant with two equal rows vanishes
// ---------------------------------------------------------------------------
//
// Swapping two rows negates a determinant, so a determinant with two equal rows
// is its own negation; in an arbitrary ring that only gives `2*det = 0`.  The
// vanishing instead comes from the pairwise cancellation of the expansion
// terms: the term of the two equal rows at columns (x, y) is the negation of
// the term at (y, x), so the double sum over ordered pairs of distinct columns
// cancels pair by pair.

/// A double sum over ordered pairs of distinct elements of an antisymmetric
/// function vanishes.
theorem sum_double_antisym_zero[R: Ring, I](cols: List[I], f: I -> I -> R) {
    cols.is_unique and (forall(x: I, y: I) {
        cols.contains(x) and cols.remove_one(x).contains(y) implies f(x, y) = -(f(y, x))
    }) implies
    sum[R](map[I, R](cols, function(x: I) {
        sum[R](map[I, R](cols.remove_one(x), function(y: I) { f(x, y) }))
    })) = R.0
} by {
    define p(l: List[I]) -> Bool {
        l.is_unique and (forall(x: I, y: I) {
            l.contains(x) and l.remove_one(x).contains(y) implies f(x, y) = -(f(y, x))
        }) implies
        sum[R](map[I, R](l, function(x: I) {
            sum[R](map[I, R](l.remove_one(x), function(y: I) { f(x, y) }))
        })) = R.0
    }
    // The empty list: both sums are empty.
    if List.nil[I].is_unique and (forall(x: I, y: I) {
        List.nil[I].contains(x) and List.nil[I].remove_one(x).contains(y) implies f(x, y) = -(f(y, x))
    }) {
        map[I, R](List.nil[I], function(x: I) {
            sum[R](map[I, R](List.nil[I].remove_one(x), function(y: I) { f(x, y) }))
        }) = List.nil[R]
        sum[R](List.nil[R]) = R.0
        sum[R](map[I, R](List.nil[I], function(x: I) {
            sum[R](map[I, R](List.nil[I].remove_one(x), function(y: I) { f(x, y) }))
        })) = R.0
    }
    p(List.nil[I])
    forall(head: I, tail: List[I]) {
        if p(tail) {
            if List.cons(head, tail).is_unique and (forall(x: I, y: I) {
                List.cons(head, tail).contains(x) and List.cons(head, tail).remove_one(x).contains(y) implies f(x, y) = -(f(y, x))
            }) {
                let antisym = forall(x: I, y: I) {
                    List.cons(head, tail).contains(x) and List.cons(head, tail).remove_one(x).contains(y) implies f(x, y) = -(f(y, x))
                }
                unique_cons_not_contains_head[I](head, tail)
                not tail.contains(head)
                unique_implies_tail_unique[I](head, tail)
                tail.is_unique
                // The antisymmetry restricted to the tail.
                forall(x: I, y: I) {
                    if tail.contains(x) and tail.remove_one(x).contains(y) {
                        if x = head {
                            tail.contains(head)
                            not tail.contains(head)
                            false
                        }
                        x != head
                        List.cons(head, tail).contains(x) = (head = x) or tail.contains(x)
                        List.cons(head, tail).contains(x)
                        List.cons(head, tail).remove_one(x) = List.cons(head, tail.remove_one(x))
                        List.cons(head, tail).remove_one(x) = List.cons(head, tail.remove_one(x))
                        List.cons(head, tail).remove_one(x).contains(y) =
                            (head = y) or tail.remove_one(x).contains(y)
                        tail.remove_one(x).contains(y)
                        if head = y {
                            List.cons(head, tail).remove_one(x).contains(y) = true
                            (head = y) or tail.remove_one(x).contains(y) = true
                            List.cons(head, tail).remove_one(x).contains(y) = (head = y) or tail.remove_one(x).contains(y)
                            List.cons(head, tail).remove_one(x).contains(y)
                        } else {
                            head != y
                            List.cons(head, tail.remove_one(x)).contains(y) = tail.remove_one(x).contains(y)
                            List.cons(head, tail).remove_one(x) = List.cons(head, tail.remove_one(x))
                            List.cons(head, tail).remove_one(x).contains(y) = tail.remove_one(x).contains(y)
                            List.cons(head, tail).remove_one(x).contains(y)
                        }
                        f(x, y) = -(f(y, x))
                    }
                    tail.contains(x) and tail.remove_one(x).contains(y) implies f(x, y) = -(f(y, x))
                }
                // The head-row summands pair with the tail-row summands at the head column.
                forall(y: I) {
                    if tail.contains(y) {
                        List.cons(head, tail).contains(head) = (head = head) or tail.contains(head)
                        List.cons(head, tail).contains(head)
                        List.cons(head, tail).remove_one(head) = tail
                        List.cons(head, tail).remove_one(head).contains(y) = tail.contains(y)
                        f(head, y) = -(f(y, head))
                    }
                    tail.contains(y) implies f(head, y) = -(f(y, head))
                }
                // Peel the head from the outer sum.
                sum_map_remove_one[I, R](List.cons(head, tail), head, function(x: I) {
                    sum[R](map[I, R](List.cons(head, tail).remove_one(x), function(y: I) { f(x, y) }))
                })
                List.cons(head, tail).contains(head)
                sum[R](map[I, R](List.cons(head, tail).remove_one(head), function(y: I) { f(head, y) })) +
                    sum[R](map[I, R](tail, function(x: I) {
                        sum[R](map[I, R](List.cons(head, tail).remove_one(x), function(y: I) { f(x, y) }))
                    })) = sum[R](map[I, R](List.cons(head, tail), function(x: I) {
                        sum[R](map[I, R](List.cons(head, tail).remove_one(x), function(y: I) { f(x, y) }))
                    }))
                List.cons(head, tail).remove_one(head) = tail
                sum[R](map[I, R](tail, function(y: I) { f(head, y) })) +
                    sum[R](map[I, R](tail, function(x: I) {
                        sum[R](map[I, R](List.cons(head, tail).remove_one(x), function(y: I) { f(x, y) }))
                    })) = sum[R](map[I, R](List.cons(head, tail), function(x: I) {
                        sum[R](map[I, R](List.cons(head, tail).remove_one(x), function(y: I) { f(x, y) }))
                    }))
                // For x in the tail, the inner list is cons(head, tail.remove_one(x)).
                forall(x: I) {
                    if tail.contains(x) {
                        if x = head {
                            tail.contains(head)
                            not tail.contains(head)
                            false
                        }
                        x != head
                        List.cons(head, tail).remove_one(x) = List.cons(head, tail.remove_one(x))
                    }
                }
                sum_map_of_pointwise[I, R](tail, function(x: I) {
                    sum[R](map[I, R](List.cons(head, tail).remove_one(x), function(y: I) { f(x, y) }))
                }, function(x: I) {
                    sum[R](map[I, R](List.cons(head, tail.remove_one(x)), function(y: I) { f(x, y) }))
                })
                sum[R](map[I, R](tail, function(x: I) {
                    sum[R](map[I, R](List.cons(head, tail).remove_one(x), function(y: I) { f(x, y) }))
                })) = sum[R](map[I, R](tail, function(x: I) {
                    sum[R](map[I, R](List.cons(head, tail.remove_one(x)), function(y: I) { f(x, y) }))
                }))
                sum[R](map[I, R](tail, function(y: I) { f(head, y) })) +
                    sum[R](map[I, R](tail, function(x: I) {
                        sum[R](map[I, R](List.cons(head, tail.remove_one(x)), function(y: I) { f(x, y) }))
                    })) = sum[R](map[I, R](List.cons(head, tail), function(x: I) {
                        sum[R](map[I, R](List.cons(head, tail).remove_one(x), function(y: I) { f(x, y) }))
                    }))
                // The inner sum splits as f(x, head) + sum over tail minus x.
                forall(x: I) {
                    if tail.contains(x) {
                        sum_map_remove_one[I, R](List.cons(head, tail.remove_one(x)), head, function(y: I) { f(x, y) })
                        List.cons(head, tail.remove_one(x)).contains(head)
                        f(x, head) + sum[R](map[I, R](List.cons(head, tail.remove_one(x)).remove_one(head), function(y: I) { f(x, y) })) =
                            sum[R](map[I, R](List.cons(head, tail.remove_one(x)), function(y: I) { f(x, y) }))
                        List.cons(head, tail.remove_one(x)).remove_one(head) = tail.remove_one(x)
                        f(x, head) + sum[R](map[I, R](tail.remove_one(x), function(y: I) { f(x, y) })) =
                            sum[R](map[I, R](List.cons(head, tail.remove_one(x)), function(y: I) { f(x, y) }))
                    }
                }
                sum_map_of_pointwise[I, R](tail, function(x: I) {
                    sum[R](map[I, R](List.cons(head, tail.remove_one(x)), function(y: I) { f(x, y) }))
                }, function(x: I) {
                    f(x, head) + sum[R](map[I, R](tail.remove_one(x), function(y: I) { f(x, y) }))
                })
                sum[R](map[I, R](tail, function(x: I) {
                    sum[R](map[I, R](List.cons(head, tail.remove_one(x)), function(y: I) { f(x, y) }))
                })) = sum[R](map[I, R](tail, function(x: I) {
                    f(x, head) + sum[R](map[I, R](tail.remove_one(x), function(y: I) { f(x, y) }))
                }))
                sum[R](map[I, R](tail, function(y: I) { f(head, y) })) +
                    sum[R](map[I, R](tail, function(x: I) {
                        f(x, head) + sum[R](map[I, R](tail.remove_one(x), function(y: I) { f(x, y) }))
                    })) = sum[R](map[I, R](List.cons(head, tail), function(x: I) {
                        sum[R](map[I, R](List.cons(head, tail).remove_one(x), function(y: I) { f(x, y) }))
                    }))
                // The sum of a pointwise sum splits.
                sum_map_add_fn[I, R](tail, function(x: I) { f(x, head) }, function(x: I) {
                    sum[R](map[I, R](tail.remove_one(x), function(y: I) { f(x, y) }))
                })
                sum[R](map[I, R](tail, function(x: I) {
                    f(x, head) + sum[R](map[I, R](tail.remove_one(x), function(y: I) { f(x, y) }))
                })) = sum[R](map[I, R](tail, function(x: I) { f(x, head) })) +
                    sum[R](map[I, R](tail, function(x: I) {
                        sum[R](map[I, R](tail.remove_one(x), function(y: I) { f(x, y) }))
                    }))
                sum[R](map[I, R](tail, function(y: I) { f(head, y) })) +
                    (sum[R](map[I, R](tail, function(x: I) { f(x, head) })) +
                        sum[R](map[I, R](tail, function(x: I) {
                            sum[R](map[I, R](tail.remove_one(x), function(y: I) { f(x, y) }))
                        }))) = sum[R](map[I, R](List.cons(head, tail), function(x: I) {
                            sum[R](map[I, R](List.cons(head, tail).remove_one(x), function(y: I) { f(x, y) }))
                        }))
                // The tail double sum vanishes by the induction hypothesis.
                p(tail) = (tail.is_unique and (forall(x: I, y: I) {
                    tail.contains(x) and tail.remove_one(x).contains(y) implies f(x, y) = -(f(y, x))
                }) implies
                sum[R](map[I, R](tail, function(x: I) {
                    sum[R](map[I, R](tail.remove_one(x), function(y: I) { f(x, y) }))
                })) = R.0)
                forall(x: I, y: I) {
                    tail.contains(x) and tail.remove_one(x).contains(y) implies f(x, y) = -(f(y, x))
                }
                tail.is_unique and (forall(x: I, y: I) {
                    tail.contains(x) and tail.remove_one(x).contains(y) implies f(x, y) = -(f(y, x))
                })
                sum[R](map[I, R](tail, function(x: I) {
                    sum[R](map[I, R](tail.remove_one(x), function(y: I) { f(x, y) }))
                })) = R.0
                // The head pairing: sum over y of f(head, y) + f(y, head) = 0.
                forall(y: I) {
                    if tail.contains(y) {
                        f(head, y) = -(f(y, head))
                    }
                    tail.contains(y) implies f(head, y) = -(f(y, head))
                }
                sum_map_of_pointwise[I, R](tail,
                    function(y: I) { f(head, y) },
                    function(y: I) { -(f(y, head)) })
                sum[R](map[I, R](tail, function(y: I) { f(head, y) })) =
                    sum[R](map[I, R](tail, function(y: I) { -(f(y, head)) }))
                sum_map_neg[I, R](tail, function(y: I) { f(y, head) })
                sum[R](map[I, R](tail, function(y: I) { -(f(y, head)) })) =
                    -(sum[R](map[I, R](tail, function(y: I) { f(y, head) })))
                sum[R](map[I, R](tail, function(y: I) { f(head, y) })) =
                    -(sum[R](map[I, R](tail, function(y: I) { f(y, head) })))
                sum[R](map[I, R](tail, function(y: I) { f(head, y) })) +
                    sum[R](map[I, R](tail, function(y: I) { f(y, head) })) =
                    -(sum[R](map[I, R](tail, function(y: I) { f(y, head) }))) +
                        sum[R](map[I, R](tail, function(y: I) { f(y, head) }))
                -(sum[R](map[I, R](tail, function(y: I) { f(y, head) }))) +
                    sum[R](map[I, R](tail, function(y: I) { f(y, head) })) = R.0
                sum[R](map[I, R](tail, function(y: I) { f(head, y) })) +
                    sum[R](map[I, R](tail, function(y: I) { f(y, head) })) = R.0
                sum[R](map[I, R](tail, function(y: I) { f(head, y) })) +
                    (sum[R](map[I, R](tail, function(x: I) { f(x, head) })) + R.0) =
                    sum[R](map[I, R](tail, function(y: I) { f(head, y) })) +
                        sum[R](map[I, R](tail, function(x: I) { f(x, head) }))
                sum[R](map[I, R](tail, function(y: I) { f(head, y) })) +
                    (sum[R](map[I, R](tail, function(x: I) { f(x, head) })) + R.0) = R.0
                sum[R](map[I, R](tail, function(y: I) { f(head, y) })) +
                    (sum[R](map[I, R](tail, function(x: I) { f(x, head) })) +
                        sum[R](map[I, R](tail, function(x: I) {
                            sum[R](map[I, R](tail.remove_one(x), function(y: I) { f(x, y) }))
                        }))) = R.0
                sum[R](map[I, R](List.cons(head, tail), function(x: I) {
                    sum[R](map[I, R](List.cons(head, tail).remove_one(x), function(y: I) { f(x, y) }))
                })) = R.0
            }
            p(tail) implies p(List.cons(head, tail))
        }
    }
    p(List.nil[I]) and forall(head: I, tail: List[I]) { p(tail) implies p(List.cons(head, tail)) }
    List.induction(p)
    forall(l: List[I]) { p(l) }
    p(cols)
    if cols.is_unique and (forall(x: I, y: I) {
        cols.contains(x) and cols.remove_one(x).contains(y) implies f(x, y) = -(f(y, x))
    }) {
        sum[R](map[I, R](cols, function(x: I) {
            sum[R](map[I, R](cols.remove_one(x), function(y: I) { f(x, y) }))
        })) = R.0
    }
}

/// A two-row expansion term with equal rows is symmetric in the two rows.
theorem det_two_row_term_equal_rows_sym[R: CommRing, I](
    entry: (I, I) -> R, r1: I, r2: I, rs: List[I], cols: List[I], x: I, y: I
) {
    (forall(c: I) { entry(r1, c) = entry(r2, c) }) implies
        det_two_row_term[R, I](entry, r1, r2, rs, cols, x, y) =
            det_two_row_term[R, I](entry, r2, r1, rs, cols, x, y)
} by {
    if forall(c: I) { entry(r1, c) = entry(r2, c) } {
        det_two_row_term[R, I](entry, r1, r2, rs, cols, x, y) =
            entry(r1, x) * alternating_sign[R](list_index_or_zero[I](cols, x)) *
                entry(r2, y) * alternating_sign[R](list_index_or_zero[I](cols.remove_one(x), y)) *
                determinant_list[R, I](entry, rs, cols.remove_one(x).remove_one(y))
        det_two_row_term[R, I](entry, r2, r1, rs, cols, x, y) =
            entry(r2, x) * alternating_sign[R](list_index_or_zero[I](cols, x)) *
                entry(r1, y) * alternating_sign[R](list_index_or_zero[I](cols.remove_one(x), y)) *
                determinant_list[R, I](entry, rs, cols.remove_one(x).remove_one(y))
        entry(r1, x) = entry(r2, x)
        entry(r1, y) = entry(r2, y)
        entry(r1, x) * alternating_sign[R](list_index_or_zero[I](cols, x)) *
            entry(r2, y) * alternating_sign[R](list_index_or_zero[I](cols.remove_one(x), y)) *
            determinant_list[R, I](entry, rs, cols.remove_one(x).remove_one(y)) =
            entry(r2, x) * alternating_sign[R](list_index_or_zero[I](cols, x)) *
                entry(r1, y) * alternating_sign[R](list_index_or_zero[I](cols.remove_one(x), y)) *
                determinant_list[R, I](entry, rs, cols.remove_one(x).remove_one(y))
        det_two_row_term[R, I](entry, r1, r2, rs, cols, x, y) =
            det_two_row_term[R, I](entry, r2, r1, rs, cols, x, y)
    }
}

/// A determinant-list whose first two rows are pointwise equal vanishes.
theorem determinant_list_equal_first_two_rows[R: CommRing, I](
    entry: (I, I) -> R, r1: I, r2: I, rs: List[I], cols: List[I]
) {
    cols.is_unique and (forall(c: I) { entry(r1, c) = entry(r2, c) }) implies
        determinant_list[R, I](entry, List.cons(r1, List.cons(r2, rs)), cols) = R.0
} by {
    if cols.is_unique and (forall(c: I) { entry(r1, c) = entry(r2, c) }) {
        let eq_rows = forall(c: I) { entry(r1, c) = entry(r2, c) }
        determinant_list_two_rows[R, I](entry, r1, r2, rs, cols)
        determinant_list[R, I](entry, List.cons(r1, List.cons(r2, rs)), cols) =
            sum[R](map[I, R](cols, function(x: I) {
                sum[R](map[I, R](cols.remove_one(x), function(y: I) {
                    det_two_row_term[R, I](entry, r1, r2, rs, cols, x, y)
                }))
            }))
        // The two-row terms are antisymmetric under (x, y) -> (y, x).
        forall(x: I, y: I) {
            if cols.contains(x) and cols.remove_one(x).contains(y) {
                det_two_row_term_equal_rows_sym[R, I](entry, r1, r2, rs, cols, x, y)
                det_two_row_term[R, I](entry, r1, r2, rs, cols, x, y) =
                    det_two_row_term[R, I](entry, r2, r1, rs, cols, x, y)
                swap_term_pair_neg[R, I](entry, r1, r2, rs, cols, x, y)
                det_two_row_term[R, I](entry, r2, r1, rs, cols, x, y) =
                    -(det_two_row_term[R, I](entry, r1, r2, rs, cols, y, x))
                det_two_row_term[R, I](entry, r1, r2, rs, cols, x, y) =
                    -(det_two_row_term[R, I](entry, r1, r2, rs, cols, y, x))
            }
            (cols.contains(x) and cols.remove_one(x).contains(y)) implies det_two_row_term[R, I](entry, r1, r2, rs, cols, x, y) =
                -(det_two_row_term[R, I](entry, r1, r2, rs, cols, y, x))
        }
        sum_double_antisym_zero[R, I](cols, function(x: I, y: I) {
            det_two_row_term[R, I](entry, r1, r2, rs, cols, x, y)
        })
        sum[R](map[I, R](cols, function(x: I) {
            sum[R](map[I, R](cols.remove_one(x), function(y: I) {
                det_two_row_term[R, I](entry, r1, r2, rs, cols, x, y)
            }))
        })) = R.0
        determinant_list[R, I](entry, List.cons(r1, List.cons(r2, rs)), cols) = R.0
    }
}

/// Swapping two adjacent rows anywhere in the row list negates a
/// determinant-list: rows deeper than the pair are reached by peeling the rows
/// before them through the head-row expansion.
theorem determinant_list_swap_adjacent[R: CommRing, I](
    entry: (I, I) -> R, prefix: List[I], r1: I, r2: I, rest: List[I], cols: List[I]
) {
    cols.is_unique implies
    determinant_list[R, I](entry, prefix + List.cons(r1, List.cons(r2, rest)), cols) =
        -(determinant_list[R, I](entry, prefix + List.cons(r2, List.cons(r1, rest)), cols))
} by {
    define p(l: List[I]) -> Bool {
        forall(cols2: List[I]) {
            cols2.is_unique implies
            determinant_list[R, I](entry, l + List.cons(r1, List.cons(r2, rest)), cols2) =
                -(determinant_list[R, I](entry, l + List.cons(r2, List.cons(r1, rest)), cols2))
        }
    }
    // Base: no rows before the pair.
    forall(cols2: List[I]) {
        if cols2.is_unique {
            determinant_list_swap_first_two_rows[R, I](entry, r1, r2, rest, cols2)
            determinant_list[R, I](entry, List.cons(r1, List.cons(r2, rest)), cols2) =
                -(determinant_list[R, I](entry, List.cons(r2, List.cons(r1, rest)), cols2))
            List.nil[I] + List.cons(r1, List.cons(r2, rest)) = List.cons(r1, List.cons(r2, rest))
            List.nil[I] + List.cons(r2, List.cons(r1, rest)) = List.cons(r2, List.cons(r1, rest))
            determinant_list[R, I](entry, List.nil[I] + List.cons(r1, List.cons(r2, rest)), cols2) =
                determinant_list[R, I](entry, List.cons(r1, List.cons(r2, rest)), cols2)
            determinant_list[R, I](entry, List.nil[I] + List.cons(r2, List.cons(r1, rest)), cols2) =
                determinant_list[R, I](entry, List.cons(r2, List.cons(r1, rest)), cols2)
            determinant_list[R, I](entry, List.nil[I] + List.cons(r1, List.cons(r2, rest)), cols2) =
                -(determinant_list[R, I](entry, List.nil[I] + List.cons(r2, List.cons(r1, rest)), cols2))
        }
    }
    p(List.nil[I])
    forall(head: I, tail: List[I]) {
        if p(tail) {
            forall(cols2: List[I]) {
                if cols2.is_unique {
                    determinant_list_cons_rows[R, I](entry, head, tail + List.cons(r1, List.cons(r2, rest)), cols2)
                    determinant_list[R, I](entry, List.cons(head, tail + List.cons(r1, List.cons(r2, rest))), cols2) =
                        sum[R](map[I, R](cols2, function(col: I) {
                            entry(head, col) * alternating_sign[R](list_index_or_zero[I](cols2, col)) *
                                determinant_list[R, I](entry, tail + List.cons(r1, List.cons(r2, rest)), cols2.remove_one(col))
                        }))
                    determinant_list_cons_rows[R, I](entry, head, tail + List.cons(r2, List.cons(r1, rest)), cols2)
                    determinant_list[R, I](entry, List.cons(head, tail + List.cons(r2, List.cons(r1, rest))), cols2) =
                        sum[R](map[I, R](cols2, function(col: I) {
                            entry(head, col) * alternating_sign[R](list_index_or_zero[I](cols2, col)) *
                                determinant_list[R, I](entry, tail + List.cons(r2, List.cons(r1, rest)), cols2.remove_one(col))
                        }))
                    forall(col: I) {
                        if cols2.contains(col) {
                            p(tail) = (forall(cols3: List[I]) {
                                cols3.is_unique implies
                                determinant_list[R, I](entry, tail + List.cons(r1, List.cons(r2, rest)), cols3) =
                                    -(determinant_list[R, I](entry, tail + List.cons(r2, List.cons(r1, rest)), cols3))
                            })
                            remove_one_unique[I](cols2, col)
                            cols2.remove_one(col).is_unique
                            determinant_list[R, I](entry, tail + List.cons(r1, List.cons(r2, rest)), cols2.remove_one(col)) =
                                -(determinant_list[R, I](entry, tail + List.cons(r2, List.cons(r1, rest)), cols2.remove_one(col)))
                        }
                        cols2.contains(col) implies determinant_list[R, I](entry, tail + List.cons(r1, List.cons(r2, rest)), cols2.remove_one(col)) =
                            -(determinant_list[R, I](entry, tail + List.cons(r2, List.cons(r1, rest)), cols2.remove_one(col)))
                        cols2.contains(col) implies entry(head, col) * alternating_sign[R](list_index_or_zero[I](cols2, col)) *
                            determinant_list[R, I](entry, tail + List.cons(r1, List.cons(r2, rest)), cols2.remove_one(col)) =
                            -(entry(head, col) * alternating_sign[R](list_index_or_zero[I](cols2, col)) *
                                determinant_list[R, I](entry, tail + List.cons(r2, List.cons(r1, rest)), cols2.remove_one(col)))
                    }
                    sum_map_of_pointwise[I, R](cols2, function(col: I) {
                        entry(head, col) * alternating_sign[R](list_index_or_zero[I](cols2, col)) *
                            determinant_list[R, I](entry, tail + List.cons(r1, List.cons(r2, rest)), cols2.remove_one(col))
                    }, function(col: I) {
                        -(entry(head, col) * alternating_sign[R](list_index_or_zero[I](cols2, col)) *
                            determinant_list[R, I](entry, tail + List.cons(r2, List.cons(r1, rest)), cols2.remove_one(col)))
                    })
                    sum[R](map[I, R](cols2, function(col: I) {
                        entry(head, col) * alternating_sign[R](list_index_or_zero[I](cols2, col)) *
                            determinant_list[R, I](entry, tail + List.cons(r1, List.cons(r2, rest)), cols2.remove_one(col))
                    })) = sum[R](map[I, R](cols2, function(col: I) {
                        -(entry(head, col) * alternating_sign[R](list_index_or_zero[I](cols2, col)) *
                            determinant_list[R, I](entry, tail + List.cons(r2, List.cons(r1, rest)), cols2.remove_one(col)))
                    }))
                    sum_map_neg[I, R](cols2, function(col: I) {
                        entry(head, col) * alternating_sign[R](list_index_or_zero[I](cols2, col)) *
                            determinant_list[R, I](entry, tail + List.cons(r2, List.cons(r1, rest)), cols2.remove_one(col))
                    })
                    sum[R](map[I, R](cols2, function(col: I) {
                        -(entry(head, col) * alternating_sign[R](list_index_or_zero[I](cols2, col)) *
                            determinant_list[R, I](entry, tail + List.cons(r2, List.cons(r1, rest)), cols2.remove_one(col)))
                    })) = -(sum[R](map[I, R](cols2, function(col: I) {
                        entry(head, col) * alternating_sign[R](list_index_or_zero[I](cols2, col)) *
                            determinant_list[R, I](entry, tail + List.cons(r2, List.cons(r1, rest)), cols2.remove_one(col))
                    })))
                    determinant_list[R, I](entry, List.cons(head, tail + List.cons(r1, List.cons(r2, rest))), cols2) =
                        -(sum[R](map[I, R](cols2, function(col: I) {
                            entry(head, col) * alternating_sign[R](list_index_or_zero[I](cols2, col)) *
                                determinant_list[R, I](entry, tail + List.cons(r2, List.cons(r1, rest)), cols2.remove_one(col))
                        })))
                    determinant_list[R, I](entry, List.cons(head, tail + List.cons(r1, List.cons(r2, rest))), cols2) =
                        -(determinant_list[R, I](entry, List.cons(head, tail + List.cons(r2, List.cons(r1, rest))), cols2))
                }
            }
            p(List.cons(head, tail)) = (forall(cols3: List[I]) {
                cols3.is_unique implies
                determinant_list[R, I](entry, List.cons(head, tail) + List.cons(r1, List.cons(r2, rest)), cols3) =
                    -(determinant_list[R, I](entry, List.cons(head, tail) + List.cons(r2, List.cons(r1, rest)), cols3))
            })
            List.cons(head, tail) + List.cons(r1, List.cons(r2, rest)) = List.cons(head, tail + List.cons(r1, List.cons(r2, rest)))
            List.cons(head, tail) + List.cons(r2, List.cons(r1, rest)) = List.cons(head, tail + List.cons(r2, List.cons(r1, rest)))
            forall(cols3: List[I]) {
                if cols3.is_unique {
                    determinant_list[R, I](entry, List.cons(head, tail) + List.cons(r1, List.cons(r2, rest)), cols3) =
                        determinant_list[R, I](entry, List.cons(head, tail + List.cons(r1, List.cons(r2, rest))), cols3)
                    determinant_list[R, I](entry, List.cons(head, tail) + List.cons(r2, List.cons(r1, rest)), cols3) =
                        determinant_list[R, I](entry, List.cons(head, tail + List.cons(r2, List.cons(r1, rest))), cols3)
                    determinant_list[R, I](entry, List.cons(head, tail + List.cons(r1, List.cons(r2, rest))), cols3) =
                        -(determinant_list[R, I](entry, List.cons(head, tail + List.cons(r2, List.cons(r1, rest))), cols3))
                    determinant_list[R, I](entry, List.cons(head, tail) + List.cons(r1, List.cons(r2, rest)), cols3) =
                        -(determinant_list[R, I](entry, List.cons(head, tail) + List.cons(r2, List.cons(r1, rest)), cols3))
                }
            }
            p(List.cons(head, tail))
        }
        p(tail) implies p(List.cons(head, tail))
    }
    p(List.nil[I]) and forall(head: I, tail: List[I]) { p(tail) implies p(List.cons(head, tail)) }
    List.induction(p)
    forall(l: List[I]) { p(l) }
    p(prefix)
    if cols.is_unique {
        determinant_list[R, I](entry, prefix + List.cons(r1, List.cons(r2, rest)), cols) =
            -(determinant_list[R, I](entry, prefix + List.cons(r2, List.cons(r1, rest)), cols))
    }
}

/// Moving a row to the front of a row list negates the determinant once per
/// row crossed.
theorem determinant_list_move_to_front[R: CommRing, I](
    entry: (I, I) -> R, prefix: List[I], r: I, suffix: List[I], cols: List[I]
) {
    cols.is_unique implies
    determinant_list[R, I](entry, prefix + List.cons(r, suffix), cols) =
        alternating_sign[R](prefix.length) * determinant_list[R, I](entry, List.cons(r, prefix + suffix), cols)
} by {
    define p(l: List[I]) -> Bool {
        forall(cols2: List[I]) {
            cols2.is_unique implies
            determinant_list[R, I](entry, l + List.cons(r, suffix), cols2) =
                alternating_sign[R](l.length) * determinant_list[R, I](entry, List.cons(r, l + suffix), cols2)
        }
    }
    // Base: nothing before the row.
    forall(cols2: List[I]) {
        if cols2.is_unique {
            List.nil[I] + List.cons(r, suffix) = List.cons(r, suffix)
            List.nil[I] + suffix = suffix
            List.cons(r, List.nil[I] + suffix) = List.cons(r, suffix)
            alternating_sign_zero[R]
            alternating_sign[R](Nat.0) = R.1
            List.nil[I].length = Nat.0
            R.1 * determinant_list[R, I](entry, List.cons(r, suffix), cols2) = determinant_list[R, I](entry, List.cons(r, suffix), cols2)
            determinant_list[R, I](entry, List.nil[I] + List.cons(r, suffix), cols2) =
                alternating_sign[R](List.nil[I].length) * determinant_list[R, I](entry, List.cons(r, List.nil[I] + suffix), cols2)
        }
    }
    p(List.nil[I])
    forall(head: I, tail: List[I]) {
        if p(tail) {
            forall(cols2: List[I]) {
                if cols2.is_unique {
                    // The left side expands along its head row `head`.
                    determinant_list_cons_rows[R, I](entry, head, tail + List.cons(r, suffix), cols2)
                    determinant_list[R, I](entry, List.cons(head, tail + List.cons(r, suffix)), cols2) =
                        sum[R](map[I, R](cols2, function(col: I) {
                            entry(head, col) * alternating_sign[R](list_index_or_zero[I](cols2, col)) *
                                determinant_list[R, I](entry, tail + List.cons(r, suffix), cols2.remove_one(col))
                        }))
                    List.cons(head, tail) + List.cons(r, suffix) = List.cons(head, tail + List.cons(r, suffix))
                    determinant_list[R, I](entry, List.cons(head, tail) + List.cons(r, suffix), cols2) =
                        sum[R](map[I, R](cols2, function(col: I) {
                            entry(head, col) * alternating_sign[R](list_index_or_zero[I](cols2, col)) *
                                determinant_list[R, I](entry, tail + List.cons(r, suffix), cols2.remove_one(col))
                        }))
                    // The induction hypothesis rewrites each residual.
                    forall(col: I) {
                        if cols2.contains(col) {
                            p(tail) = (forall(cols3: List[I]) {
                                cols3.is_unique implies
                                determinant_list[R, I](entry, tail + List.cons(r, suffix), cols3) =
                                    alternating_sign[R](tail.length) * determinant_list[R, I](entry, List.cons(r, tail + suffix), cols3)
                            })
                            remove_one_unique[I](cols2, col)
                            cols2.remove_one(col).is_unique
                            determinant_list[R, I](entry, tail + List.cons(r, suffix), cols2.remove_one(col)) =
                                alternating_sign[R](tail.length) * determinant_list[R, I](entry, List.cons(r, tail + suffix), cols2.remove_one(col))
                        }
                        cols2.contains(col) implies determinant_list[R, I](entry, tail + List.cons(r, suffix), cols2.remove_one(col)) =
                            alternating_sign[R](tail.length) * determinant_list[R, I](entry, List.cons(r, tail + suffix), cols2.remove_one(col))
                        cols2.contains(col) implies entry(head, col) * alternating_sign[R](list_index_or_zero[I](cols2, col)) *
                            determinant_list[R, I](entry, tail + List.cons(r, suffix), cols2.remove_one(col)) =
                            entry(head, col) * alternating_sign[R](list_index_or_zero[I](cols2, col)) *
                                alternating_sign[R](tail.length) *
                                determinant_list[R, I](entry, List.cons(r, tail + suffix), cols2.remove_one(col))
                    }
                    sum_map_of_pointwise[I, R](cols2, function(col: I) {
                        entry(head, col) * alternating_sign[R](list_index_or_zero[I](cols2, col)) *
                            determinant_list[R, I](entry, tail + List.cons(r, suffix), cols2.remove_one(col))
                    }, function(col: I) {
                        entry(head, col) * alternating_sign[R](list_index_or_zero[I](cols2, col)) *
                            alternating_sign[R](tail.length) * determinant_list[R, I](entry, List.cons(r, tail + suffix), cols2.remove_one(col))
                    })
                    sum[R](map[I, R](cols2, function(col: I) {
                        entry(head, col) * alternating_sign[R](list_index_or_zero[I](cols2, col)) *
                            determinant_list[R, I](entry, tail + List.cons(r, suffix), cols2.remove_one(col))
                    })) = sum[R](map[I, R](cols2, function(col: I) {
                        entry(head, col) * alternating_sign[R](list_index_or_zero[I](cols2, col)) *
                            alternating_sign[R](tail.length) * determinant_list[R, I](entry, List.cons(r, tail + suffix), cols2.remove_one(col))
                    }))
                    // Pull the sign of the residual out of the sum.
                    forall(col: I) {
                        if cols2.contains(col) {
                            entry(head, col) * alternating_sign[R](list_index_or_zero[I](cols2, col)) *
                                alternating_sign[R](tail.length) * determinant_list[R, I](entry, List.cons(r, tail + suffix), cols2.remove_one(col)) =
                                alternating_sign[R](tail.length) *
                                    (entry(head, col) * alternating_sign[R](list_index_or_zero[I](cols2, col)) *
                                        determinant_list[R, I](entry, List.cons(r, tail + suffix), cols2.remove_one(col)))
                        }
                    }
                    sum_map_of_pointwise[I, R](cols2, function(col: I) {
                        entry(head, col) * alternating_sign[R](list_index_or_zero[I](cols2, col)) *
                            alternating_sign[R](tail.length) * determinant_list[R, I](entry, List.cons(r, tail + suffix), cols2.remove_one(col))
                    }, function(col: I) {
                        alternating_sign[R](tail.length) *
                            (entry(head, col) * alternating_sign[R](list_index_or_zero[I](cols2, col)) *
                                determinant_list[R, I](entry, List.cons(r, tail + suffix), cols2.remove_one(col)))
                    })
                    sum[R](map[I, R](cols2, function(col: I) {
                        entry(head, col) * alternating_sign[R](list_index_or_zero[I](cols2, col)) *
                            alternating_sign[R](tail.length) * determinant_list[R, I](entry, List.cons(r, tail + suffix), cols2.remove_one(col))
                    })) = sum[R](map[I, R](cols2, function(col: I) {
                        alternating_sign[R](tail.length) *
                            (entry(head, col) * alternating_sign[R](list_index_or_zero[I](cols2, col)) *
                                determinant_list[R, I](entry, List.cons(r, tail + suffix), cols2.remove_one(col)))
                    }))
                    sum_map_scalar_mul_semiring_left[I, R](cols2, alternating_sign[R](tail.length),
                        function(col: I) {
                            entry(head, col) * alternating_sign[R](list_index_or_zero[I](cols2, col)) *
                                determinant_list[R, I](entry, List.cons(r, tail + suffix), cols2.remove_one(col))
                        })
                    alternating_sign[R](tail.length) *
                        sum[R](map[I, R](cols2, function(col: I) {
                            entry(head, col) * alternating_sign[R](list_index_or_zero[I](cols2, col)) *
                                determinant_list[R, I](entry, List.cons(r, tail + suffix), cols2.remove_one(col))
                        })) = sum[R](map[I, R](cols2, function(col: I) {
                            alternating_sign[R](tail.length) *
                                (entry(head, col) * alternating_sign[R](list_index_or_zero[I](cols2, col)) *
                                    determinant_list[R, I](entry, List.cons(r, tail + suffix), cols2.remove_one(col)))
                        }))
                    // The right side: det([r] ++ head :: tail ++ suffix) is the negation of
                    // the head-row expansion of [head, r] ++ tail ++ suffix.
                    determinant_list_swap_first_two_rows[R, I](entry, head, r, tail + suffix, cols2)
                    determinant_list[R, I](entry, List.cons(r, List.cons(head, tail + suffix)), cols2) =
                        -(determinant_list[R, I](entry, List.cons(head, List.cons(r, tail + suffix)), cols2))
                    determinant_list_cons_rows[R, I](entry, head, List.cons(r, tail + suffix), cols2)
                    determinant_list[R, I](entry, List.cons(head, List.cons(r, tail + suffix)), cols2) =
                        sum[R](map[I, R](cols2, function(col: I) {
                            entry(head, col) * alternating_sign[R](list_index_or_zero[I](cols2, col)) *
                                determinant_list[R, I](entry, List.cons(r, tail + suffix), cols2.remove_one(col))
                        }))
                    determinant_list[R, I](entry, List.cons(r, List.cons(head, tail + suffix)), cols2) =
                        -(sum[R](map[I, R](cols2, function(col: I) {
                            entry(head, col) * alternating_sign[R](list_index_or_zero[I](cols2, col)) *
                                determinant_list[R, I](entry, List.cons(r, tail + suffix), cols2.remove_one(col))
                        })))
                    List.cons(r, List.cons(head, tail) + suffix) = List.cons(r, List.cons(head, tail + suffix))
                    alternating_sign_suc[R](tail.length)
                    alternating_sign[R](tail.length.suc) = -alternating_sign[R](tail.length)
                    List.cons(head, tail).length = tail.length.suc
                    alternating_sign[R](List.cons(head, tail).length) *
                        determinant_list[R, I](entry, List.cons(r, List.cons(head, tail) + suffix), cols2) =
                        alternating_sign[R](tail.length.suc) *
                            (-(sum[R](map[I, R](cols2, function(col: I) {
                                entry(head, col) * alternating_sign[R](list_index_or_zero[I](cols2, col)) *
                                    determinant_list[R, I](entry, List.cons(r, tail + suffix), cols2.remove_one(col))
                            }))))
                    alternating_sign[R](tail.length.suc) *
                        (-(sum[R](map[I, R](cols2, function(col: I) {
                            entry(head, col) * alternating_sign[R](list_index_or_zero[I](cols2, col)) *
                                determinant_list[R, I](entry, List.cons(r, tail + suffix), cols2.remove_one(col))
                        })))) =
                        alternating_sign[R](tail.length) *
                            sum[R](map[I, R](cols2, function(col: I) {
                                entry(head, col) * alternating_sign[R](list_index_or_zero[I](cols2, col)) *
                                    determinant_list[R, I](entry, List.cons(r, tail + suffix), cols2.remove_one(col))
                            }))
                    // The two negated signs cancel: (-alt) * (-S) = alt * S.
                    alternating_sign[R](List.cons(head, tail).length) *
                        determinant_list[R, I](entry, List.cons(r, List.cons(head, tail) + suffix), cols2) =
                        alternating_sign[R](tail.length) *
                            sum[R](map[I, R](cols2, function(col: I) {
                                entry(head, col) * alternating_sign[R](list_index_or_zero[I](cols2, col)) *
                                    determinant_list[R, I](entry, List.cons(r, tail + suffix), cols2.remove_one(col))
                            }))
                    // The left side (with the induction hypothesis applied) is the same sum.
                    alternating_sign[R](tail.length) *
                        sum[R](map[I, R](cols2, function(col: I) {
                            entry(head, col) * alternating_sign[R](list_index_or_zero[I](cols2, col)) *
                                determinant_list[R, I](entry, List.cons(r, tail + suffix), cols2.remove_one(col))
                        })) =
                        alternating_sign[R](List.cons(head, tail).length) *
                            determinant_list[R, I](entry, List.cons(r, List.cons(head, tail) + suffix), cols2)
                    determinant_list[R, I](entry, List.cons(head, tail) + List.cons(r, suffix), cols2) =
                        alternating_sign[R](List.cons(head, tail).length) *
                            determinant_list[R, I](entry, List.cons(r, List.cons(head, tail) + suffix), cols2)
                }
            }
            p(List.cons(head, tail))
        }
        p(tail) implies p(List.cons(head, tail))
    }
    p(List.nil[I]) and forall(head: I, tail: List[I]) { p(tail) implies p(List.cons(head, tail)) }
    List.induction(p)
    forall(l: List[I]) { p(l) }
    p(prefix)
    if cols.is_unique {
        determinant_list[R, I](entry, prefix + List.cons(r, suffix), cols) =
            alternating_sign[R](prefix.length) * determinant_list[R, I](entry, List.cons(r, prefix + suffix), cols)
    }
}

/// Moving a row to the second position past a prefix changes the determinant
/// by the sign of the prefix length.
theorem det_cons_head_move_r2[R: CommRing, I](
    entry: (I, I) -> R, prefix: List[I], r1: I, r2: I, t: List[I], cols: List[I]
) {
    cols.is_unique implies
    determinant_list[R, I](entry, List.cons(r1, prefix + List.cons(r2, t)), cols) =
        alternating_sign[R](prefix.length) *
            determinant_list[R, I](entry, List.cons(r1, List.cons(r2, prefix + t)), cols)
} by {
    if cols.is_unique {
        determinant_list_cons_rows[R, I](entry, r1, prefix + List.cons(r2, t), cols)
        determinant_list[R, I](entry, List.cons(r1, prefix + List.cons(r2, t)), cols) =
            sum[R](map[I, R](cols, function(col: I) {
                entry(r1, col) * alternating_sign[R](list_index_or_zero[I](cols, col)) *
                    determinant_list[R, I](entry, prefix + List.cons(r2, t), cols.remove_one(col))
            }))
        forall(col: I) {
            if cols.contains(col) {
                determinant_list_move_to_front[R, I](entry, prefix, r2, t, cols.remove_one(col))
                remove_one_unique[I](cols, col)
                cols.remove_one(col).is_unique
                determinant_list[R, I](entry, prefix + List.cons(r2, t), cols.remove_one(col)) =
                    alternating_sign[R](prefix.length) *
                        determinant_list[R, I](entry, List.cons(r2, prefix + t), cols.remove_one(col))
            }
            cols.contains(col) implies determinant_list[R, I](entry, prefix + List.cons(r2, t), cols.remove_one(col)) =
                alternating_sign[R](prefix.length) *
                    determinant_list[R, I](entry, List.cons(r2, prefix + t), cols.remove_one(col))
            cols.contains(col) implies entry(r1, col) * alternating_sign[R](list_index_or_zero[I](cols, col)) *
                determinant_list[R, I](entry, prefix + List.cons(r2, t), cols.remove_one(col)) =
                entry(r1, col) * alternating_sign[R](list_index_or_zero[I](cols, col)) *
                    alternating_sign[R](prefix.length) *
                    determinant_list[R, I](entry, List.cons(r2, prefix + t), cols.remove_one(col))
        }
        sum_map_of_pointwise[I, R](cols, function(col: I) {
            entry(r1, col) * alternating_sign[R](list_index_or_zero[I](cols, col)) *
                determinant_list[R, I](entry, prefix + List.cons(r2, t), cols.remove_one(col))
        }, function(col: I) {
            entry(r1, col) * alternating_sign[R](list_index_or_zero[I](cols, col)) *
                alternating_sign[R](prefix.length) *
                determinant_list[R, I](entry, List.cons(r2, prefix + t), cols.remove_one(col))
        })
        sum[R](map[I, R](cols, function(col: I) {
            entry(r1, col) * alternating_sign[R](list_index_or_zero[I](cols, col)) *
                determinant_list[R, I](entry, prefix + List.cons(r2, t), cols.remove_one(col))
        })) = sum[R](map[I, R](cols, function(col: I) {
            entry(r1, col) * alternating_sign[R](list_index_or_zero[I](cols, col)) *
                alternating_sign[R](prefix.length) *
                determinant_list[R, I](entry, List.cons(r2, prefix + t), cols.remove_one(col))
        }))
        forall(col: I) {
            if cols.contains(col) {
                entry(r1, col) * alternating_sign[R](list_index_or_zero[I](cols, col)) *
                    alternating_sign[R](prefix.length) *
                    determinant_list[R, I](entry, List.cons(r2, prefix + t), cols.remove_one(col)) =
                    alternating_sign[R](prefix.length) *
                        (entry(r1, col) * alternating_sign[R](list_index_or_zero[I](cols, col)) *
                            determinant_list[R, I](entry, List.cons(r2, prefix + t), cols.remove_one(col)))
            }
        }
        sum_map_of_pointwise[I, R](cols, function(col: I) {
            entry(r1, col) * alternating_sign[R](list_index_or_zero[I](cols, col)) *
                alternating_sign[R](prefix.length) *
                determinant_list[R, I](entry, List.cons(r2, prefix + t), cols.remove_one(col))
        }, function(col: I) {
            alternating_sign[R](prefix.length) *
                (entry(r1, col) * alternating_sign[R](list_index_or_zero[I](cols, col)) *
                    determinant_list[R, I](entry, List.cons(r2, prefix + t), cols.remove_one(col)))
        })
        sum[R](map[I, R](cols, function(col: I) {
            entry(r1, col) * alternating_sign[R](list_index_or_zero[I](cols, col)) *
                alternating_sign[R](prefix.length) *
                determinant_list[R, I](entry, List.cons(r2, prefix + t), cols.remove_one(col))
        })) = sum[R](map[I, R](cols, function(col: I) {
            alternating_sign[R](prefix.length) *
                (entry(r1, col) * alternating_sign[R](list_index_or_zero[I](cols, col)) *
                    determinant_list[R, I](entry, List.cons(r2, prefix + t), cols.remove_one(col)))
        }))
        sum_map_scalar_mul_semiring_left[I, R](cols, alternating_sign[R](prefix.length),
            function(col: I) {
                entry(r1, col) * alternating_sign[R](list_index_or_zero[I](cols, col)) *
                    determinant_list[R, I](entry, List.cons(r2, prefix + t), cols.remove_one(col))
            })
        alternating_sign[R](prefix.length) *
            sum[R](map[I, R](cols, function(col: I) {
                entry(r1, col) * alternating_sign[R](list_index_or_zero[I](cols, col)) *
                    determinant_list[R, I](entry, List.cons(r2, prefix + t), cols.remove_one(col))
            })) = sum[R](map[I, R](cols, function(col: I) {
                alternating_sign[R](prefix.length) *
                    (entry(r1, col) * alternating_sign[R](list_index_or_zero[I](cols, col)) *
                        determinant_list[R, I](entry, List.cons(r2, prefix + t), cols.remove_one(col)))
            }))
        determinant_list_cons_rows[R, I](entry, r1, List.cons(r2, prefix + t), cols)
        determinant_list[R, I](entry, List.cons(r1, List.cons(r2, prefix + t)), cols) =
            sum[R](map[I, R](cols, function(col: I) {
                entry(r1, col) * alternating_sign[R](list_index_or_zero[I](cols, col)) *
                    determinant_list[R, I](entry, List.cons(r2, prefix + t), cols.remove_one(col))
            }))
        determinant_list[R, I](entry, List.cons(r1, prefix + List.cons(r2, t)), cols) =
            alternating_sign[R](prefix.length) *
                determinant_list[R, I](entry, List.cons(r1, List.cons(r2, prefix + t)), cols)
    }
}

/// A head row equal to a row deeper in the tail makes a determinant-list
/// vanish: the deeper row is moved to the second position and the equal pair
/// cancels.
theorem det_cons_head_equal_deep[R: CommRing, I](
    entry: (I, I) -> R, prefix: List[I], t: List[I], r1: I, r2: I, cols: List[I]
) {
    cols.is_unique and (prefix + t).is_unique and t.contains(r2) and
    (forall(c: I) { entry(r1, c) = entry(r2, c) })
        implies determinant_list[R, I](entry, List.cons(r1, prefix + t), cols) = R.0
} by {
    define p(l: List[I]) -> Bool {
        forall(prefix2: List[I], cols2: List[I]) {
            cols2.is_unique and (prefix2 + l).is_unique and l.contains(r2) and
            (forall(c: I) { entry(r1, c) = entry(r2, c) })
                implies determinant_list[R, I](entry, List.cons(r1, prefix2 + l), cols2) = R.0
        }
    }
    forall(prefix2: List[I], cols2: List[I]) {
        if cols2.is_unique and (prefix2 + List.nil[I]).is_unique and List.nil[I].contains(r2) and
            (forall(c: I) { entry(r1, c) = entry(r2, c) }) {
            List.nil[I].contains(r2) = false
            false
        }
    }
    p(List.nil[I])
    forall(head: I, tail: List[I]) {
        if p(tail) {
            forall(prefix2: List[I], cols2: List[I]) {
                if cols2.is_unique and (prefix2 + List.cons(head, tail)).is_unique and
                    List.cons(head, tail).contains(r2) and
                    (forall(c: I) { entry(r1, c) = entry(r2, c) }) {
                    if head = r2 {
                        det_cons_head_move_r2[R, I](entry, prefix2, r1, r2, tail, cols2)
                        determinant_list[R, I](entry, List.cons(r1, prefix2 + List.cons(r2, tail)), cols2) =
                            alternating_sign[R](prefix2.length) *
                                determinant_list[R, I](entry, List.cons(r1, List.cons(r2, prefix2 + tail)), cols2)
                        determinant_list_equal_first_two_rows[R, I](entry, r1, r2, prefix2 + tail, cols2)
                        determinant_list[R, I](entry, List.cons(r1, List.cons(r2, prefix2 + tail)), cols2) = R.0
                        alternating_sign[R](prefix2.length) *
                            determinant_list[R, I](entry, List.cons(r1, List.cons(r2, prefix2 + tail)), cols2) = R.0
                        determinant_list[R, I](entry, List.cons(r1, prefix2 + List.cons(r2, tail)), cols2) = R.0
                        determinant_list[R, I](entry, List.cons(r1, prefix2 + List.cons(head, tail)), cols2) = R.0
                    } else {
                        head != r2
                        tail.contains(r2)
                        List.singleton(head) + tail = List.cons(head, tail)
                        (prefix2 + List.singleton(head)) + tail = prefix2 + (List.singleton(head) + tail)
                        (prefix2 + List.singleton(head)) + tail = prefix2 + List.cons(head, tail)
                        List.cons(r1, (prefix2 + List.singleton(head)) + tail) =
                            List.cons(r1, prefix2 + List.cons(head, tail))
                        p(tail) = (forall(prefix3: List[I], cols3: List[I]) {
                            cols3.is_unique and (prefix3 + tail).is_unique and tail.contains(r2) and
                            (forall(c: I) { entry(r1, c) = entry(r2, c) })
                                implies determinant_list[R, I](entry, List.cons(r1, prefix3 + tail), cols3) = R.0
                        })
                        determinant_list[R, I](entry, List.cons(r1, (prefix2 + List.singleton(head)) + tail), cols2) = R.0
                        determinant_list[R, I](entry, List.cons(r1, prefix2 + List.cons(head, tail)), cols2) = R.0
                    }
                }
            }
            p(List.cons(head, tail))
        }
        p(tail) implies p(List.cons(head, tail))
    }
    p(List.nil[I]) and forall(head: I, tail: List[I]) { p(tail) implies p(List.cons(head, tail)) }
    List.induction(p)
    forall(l: List[I]) { p(l) }
    p(t)
    if cols.is_unique and (prefix + t).is_unique and t.contains(r2) and
        (forall(c: I) { entry(r1, c) = entry(r2, c) }) {
        determinant_list[R, I](entry, List.cons(r1, prefix + t), cols) = R.0
    }
}


/// A determinant-list with two pointwise equal rows anywhere in the row list
/// vanishes.
theorem determinant_list_equal_rows[R: CommRing, I](
    entry: (I, I) -> R, p1: List[I], r1: I, mid: List[I], r2: I, rest: List[I], cols: List[I]
) {
    cols.is_unique and (forall(c: I) { entry(r1, c) = entry(r2, c) }) implies
        determinant_list[R, I](entry, p1 + List.cons(r1, mid + List.cons(r2, rest)), cols) = R.0
} by {
    if cols.is_unique and (forall(c: I) { entry(r1, c) = entry(r2, c) }) {
        // Move r2 past the rows before it, leaving the equal pair at the head.
        determinant_list_move_to_front[R, I](entry, p1 + List.cons(r1, mid), r2, rest, cols)
        determinant_list[R, I](entry, (p1 + List.cons(r1, mid)) + List.cons(r2, rest), cols) =
            alternating_sign[R]((p1 + List.cons(r1, mid)).length) *
                determinant_list[R, I](entry, List.cons(r2, (p1 + List.cons(r1, mid)) + rest), cols)
        // Move r1 past the remaining prefix, making the equal pair the first two rows.
        det_cons_head_move_r2[R, I](entry, p1, r2, r1, mid + rest, cols)
        determinant_list[R, I](entry, List.cons(r2, p1 + List.cons(r1, mid + rest)), cols) =
            alternating_sign[R](p1.length) *
                determinant_list[R, I](entry, List.cons(r2, List.cons(r1, p1 + mid + rest)), cols)
        (p1 + List.cons(r1, mid)) + rest = p1 + List.cons(r1, mid + rest)
        determinant_list[R, I](entry, List.cons(r2, (p1 + List.cons(r1, mid)) + rest), cols) =
            determinant_list[R, I](entry, List.cons(r2, p1 + List.cons(r1, mid + rest)), cols)
        determinant_list[R, I](entry, List.cons(r2, (p1 + List.cons(r1, mid)) + rest), cols) =
            alternating_sign[R](p1.length) *
                determinant_list[R, I](entry, List.cons(r2, List.cons(r1, p1 + mid + rest)), cols)
        // The equal pair at the head vanishes, so the whole determinant does.
        determinant_list_equal_first_two_rows[R, I](entry, r2, r1, p1 + mid + rest, cols)
        determinant_list[R, I](entry, List.cons(r2, List.cons(r1, p1 + mid + rest)), cols) = R.0
        alternating_sign[R](p1.length) *
            determinant_list[R, I](entry, List.cons(r2, List.cons(r1, p1 + mid + rest)), cols) = R.0
        determinant_list[R, I](entry, List.cons(r2, (p1 + List.cons(r1, mid)) + rest), cols) = R.0
        determinant_list[R, I](entry, (p1 + List.cons(r1, mid)) + List.cons(r2, rest), cols) = R.0
        (p1 + List.cons(r1, mid)) + List.cons(r2, rest) = p1 + List.cons(r1, mid + List.cons(r2, rest))
        determinant_list[R, I](entry, (p1 + List.cons(r1, mid)) + List.cons(r2, rest), cols) =
            determinant_list[R, I](entry, p1 + List.cons(r1, mid + List.cons(r2, rest)), cols)
        determinant_list[R, I](entry, p1 + List.cons(r1, mid + List.cons(r2, rest)), cols) = R.0
    }
}

/// A mapped sum over a list vanishes when the function vanishes on every
/// element of the list.
theorem sum_map_zero_contains[T, A: AddCommMonoid](items: List[T], g: T -> A) {
    (forall(item: T) { items.contains(item) implies g(item) = A.0 }) implies
        sum[A](map[T, A](items, g)) = A.0
} by {
    define p(l: List[T]) -> Bool {
        (forall(item: T) { l.contains(item) implies g(item) = A.0 }) implies
            sum[A](map[T, A](l, g)) = A.0
    }
    if forall(item: T) { List.nil[T].contains(item) implies g(item) = A.0 } {
        map[T, A](List.nil[T], g) = List.nil[A]
        sum[A](List.nil[A]) = A.0
    }
    p(List.nil[T])
    forall(head: T, tail: List[T]) {
        if p(tail) {
            if forall(item: T) { List.cons(head, tail).contains(item) implies g(item) = A.0 } {
                forall(item: T) {
                    if tail.contains(item) {
                        List.cons(head, tail).contains(item) = (head = item) or tail.contains(item)
                        List.cons(head, tail).contains(item)
                        g(item) = A.0
                    }
                    tail.contains(item) implies g(item) = A.0
                }
                forall(item: T) { tail.contains(item) implies g(item) = A.0 }
                p(tail) = ((forall(item: T) { tail.contains(item) implies g(item) = A.0 }) implies
                    sum[A](map[T, A](tail, g)) = A.0)
                sum[A](map[T, A](tail, g)) = A.0
                List.cons(head, tail).contains(head) = (head = head) or tail.contains(head)
                List.cons(head, tail).contains(head)
                g(head) = A.0
                map[T, A](List.cons(head, tail), g) = List.cons(g(head), map[T, A](tail, g))
                sum[A](List.cons(g(head), map[T, A](tail, g))) = g(head) + sum[A](map[T, A](tail, g))
                sum[A](map[T, A](List.cons(head, tail), g)) = g(head) + sum[A](map[T, A](tail, g))
                g(head) + sum[A](map[T, A](tail, g)) = A.0 + sum[A](map[T, A](tail, g))
                g(head) + sum[A](map[T, A](tail, g)) = A.0 + A.0
                sum[A](map[T, A](List.cons(head, tail), g)) = A.0 + A.0
                A.0 + A.0 = A.0
                sum[A](map[T, A](List.cons(head, tail), g)) = A.0
            }
            p(List.cons(head, tail))
        }
        p(tail) implies p(List.cons(head, tail))
    }
    p(List.nil[T]) and forall(head: T, tail: List[T]) { p(tail) implies p(List.cons(head, tail)) }
    List.induction(p)
    forall(l: List[T]) { p(l) }
    p(items)
    if forall(item: T) { items.contains(item) implies g(item) = A.0 } {
        sum[A](map[T, A](items, g)) = A.0
    }
}

/// A determinant-list with two pointwise equal rows anywhere in the row list
/// vanishes, stated by membership so that the canonical enumeration applies
/// directly.
theorem determinant_list_equal_rows_mem[R: CommRing, I](
    entry: (I, I) -> R, rows: List[I], cols: List[I], r1: I, r2: I
) {
    cols.is_unique and rows.is_unique and rows.contains(r1) and rows.contains(r2) and r1 != r2 and
    (forall(c: I) { entry(r1, c) = entry(r2, c) })
        implies determinant_list[R, I](entry, rows, cols) = R.0
} by {
    define p(l: List[I]) -> Bool {
        forall(cols2: List[I]) {
            cols2.is_unique and l.is_unique and l.contains(r1) and l.contains(r2) and r1 != r2 and
            (forall(c: I) { entry(r1, c) = entry(r2, c) })
                implies determinant_list[R, I](entry, l, cols2) = R.0
        }
    }
    forall(cols2: List[I]) {
        if cols2.is_unique and List.nil[I].is_unique and List.nil[I].contains(r1) and
            List.nil[I].contains(r2) and r1 != r2 and (forall(c: I) { entry(r1, c) = entry(r2, c) }) {
            List.nil[I].contains(r1) = false
            false
        }
    }
    p(List.nil[I])
    forall(head: I, tail: List[I]) {
        if p(tail) {
            forall(cols2: List[I]) {
                if cols2.is_unique and List.cons(head, tail).is_unique and
                    List.cons(head, tail).contains(r1) and List.cons(head, tail).contains(r2) and
                    r1 != r2 and (forall(c: I) { entry(r1, c) = entry(r2, c) }) {
                    if head = r1 {
                        det_cons_head_equal_deep[R, I](entry, List.nil[I], tail, r1, r2, cols2)
                        unique_implies_tail_unique[I](head, tail)
                        tail.is_unique
                        List.nil[I] + tail = tail
                        (List.nil[I] + tail).is_unique
                        tail.contains(r2)
                        determinant_list[R, I](entry, List.cons(r1, List.nil[I] + tail), cols2) = R.0
                        List.cons(r1, List.nil[I] + tail) = List.cons(head, tail)
                        determinant_list[R, I](entry, List.cons(head, tail), cols2) = R.0
                    } else {
                        if head = r2 {
                            det_cons_head_equal_deep[R, I](entry, List.nil[I], tail, r2, r1, cols2)
                            unique_implies_tail_unique[I](head, tail)
                            tail.is_unique
                            List.nil[I] + tail = tail
                            (List.nil[I] + tail).is_unique
                            tail.contains(r1)
                            forall(c: I) {
                                entry(r2, c) = entry(r1, c)
                            }
                            determinant_list[R, I](entry, List.cons(r2, List.nil[I] + tail), cols2) = R.0
                            List.cons(r2, List.nil[I] + tail) = List.cons(head, tail)
                            determinant_list[R, I](entry, List.cons(head, tail), cols2) = R.0
                        } else {
                            head != r1
                            head != r2
                            tail.contains(r1)
                            tail.contains(r2)
                            determinant_list_cons_rows[R, I](entry, head, tail, cols2)
                            determinant_list[R, I](entry, List.cons(head, tail), cols2) =
                                sum[R](map[I, R](cols2, function(col: I) {
                                    entry(head, col) * alternating_sign[R](list_index_or_zero[I](cols2, col)) *
                                        determinant_list[R, I](entry, tail, cols2.remove_one(col))
                                }))
                            forall(col: I) {
                                if cols2.contains(col) {
                                    p(tail) = (forall(cols3: List[I]) {
                                        cols3.is_unique and tail.is_unique and tail.contains(r1) and tail.contains(r2) and r1 != r2 and
                                        (forall(c: I) { entry(r1, c) = entry(r2, c) })
                                            implies determinant_list[R, I](entry, tail, cols3) = R.0
                                    })
                                    remove_one_unique[I](cols2, col)
                                    cols2.remove_one(col).is_unique
                                    determinant_list[R, I](entry, tail, cols2.remove_one(col)) = R.0
                                }
                                cols2.contains(col) implies determinant_list[R, I](entry, tail, cols2.remove_one(col)) = R.0
                                entry(head, col) * alternating_sign[R](list_index_or_zero[I](cols2, col)) * R.0 = R.0
                                cols2.contains(col) implies entry(head, col) * alternating_sign[R](list_index_or_zero[I](cols2, col)) *
                                    determinant_list[R, I](entry, tail, cols2.remove_one(col)) =
                                    entry(head, col) * alternating_sign[R](list_index_or_zero[I](cols2, col)) * R.0
                                cols2.contains(col) implies entry(head, col) * alternating_sign[R](list_index_or_zero[I](cols2, col)) *
                                    determinant_list[R, I](entry, tail, cols2.remove_one(col)) = R.0
                            }
                            sum_map_zero_contains[I, R](cols2, function(col: I) {
                                entry(head, col) * alternating_sign[R](list_index_or_zero[I](cols2, col)) *
                                    determinant_list[R, I](entry, tail, cols2.remove_one(col))
                            })
                            sum[R](map[I, R](cols2, function(col: I) {
                                entry(head, col) * alternating_sign[R](list_index_or_zero[I](cols2, col)) *
                                    determinant_list[R, I](entry, tail, cols2.remove_one(col))
                            })) = R.0
                            determinant_list[R, I](entry, List.cons(head, tail), cols2) = R.0
                        }
                    }
                }
            }
            p(List.cons(head, tail))
        }
        p(tail) implies p(List.cons(head, tail))
    }
    p(List.nil[I]) and forall(head: I, tail: List[I]) { p(tail) implies p(List.cons(head, tail)) }
    List.induction(p)
    forall(l: List[I]) { p(l) }
    p(rows)
    if cols.is_unique and rows.is_unique and rows.contains(r1) and rows.contains(r2) and r1 != r2 and
        (forall(c: I) { entry(r1, c) = entry(r2, c) }) {
        determinant_list[R, I](entry, rows, cols) = R.0
    }
}

/// The canonical determinant of a matrix with two equal rows vanishes.
theorem matrix_det_suc_equal_rows[R: CommRing](n: Nat, a: Matrix[R, n.suc, n.suc], i: Fin[n.suc], j: Fin[n.suc]) {
    i != j and (forall(c: Fin[n.suc]) {
        matrix_entry[R](n.suc, n.suc, a, i, c) = matrix_entry[R](n.suc, n.suc, a, j, c)
    }) implies matrix_det_suc[R](n, a) = R.0
} by {
    if i != j and (forall(c: Fin[n.suc]) {
        matrix_entry[R](n.suc, n.suc, a, i, c) = matrix_entry[R](n.suc, n.suc, a, j, c)
    }) {
        matrix_det_suc_unfold[R](n, a)
        matrix_det_suc[R](n, a) = matrix_det_list[R](n.suc, a, fin_enum_suc(n), fin_enum_suc(n))
        matrix_det_list[R](n.suc, a, fin_enum_suc(n), fin_enum_suc(n)) =
            determinant_list[R, Fin[n.suc]](square_matrix_entry_function[R](n.suc, a), fin_enum_suc(n), fin_enum_suc(n))
        matrix_det_suc[R](n, a) =
            determinant_list[R, Fin[n.suc]](square_matrix_entry_function[R](n.suc, a), fin_enum_suc(n), fin_enum_suc(n))
        forall(c: Fin[n.suc]) {
            matrix_entry[R](n.suc, n.suc, a, i, c) = matrix_entry[R](n.suc, n.suc, a, j, c)
            square_matrix_entry_function[R](n.suc, a, i, c) =
                matrix_entry[R](n.suc, n.suc, a, i, c)
            square_matrix_entry_function[R](n.suc, a, j, c) =
                matrix_entry[R](n.suc, n.suc, a, j, c)
            square_matrix_entry_function[R](n.suc, a, i, c) =
                square_matrix_entry_function[R](n.suc, a, j, c)
        }
        fin_enum_suc_is_unique(n)
        fin_enum_suc(n).is_unique
        fin_enum_suc_contains(n, i)
        fin_enum_suc(n).contains(i)
        fin_enum_suc_contains(n, j)
        fin_enum_suc(n).contains(j)
        determinant_list_equal_rows_mem[R, Fin[n.suc]](square_matrix_entry_function[R](n.suc, a),
            fin_enum_suc(n), fin_enum_suc(n), i, j)
        determinant_list[R, Fin[n.suc]](square_matrix_entry_function[R](n.suc, a), fin_enum_suc(n), fin_enum_suc(n)) = R.0
        matrix_det_suc[R](n, a) = R.0
    }
}

// ---------------------------------------------------------------------------
// Characteristic polynomial basics: the determinant as the constant term
// ---------------------------------------------------------------------------
//
// The characteristic polynomial of a two-by-two matrix has the determinant as
// its constant term: substituting `lam = 0` into `(a00 - lam)(a11 - lam) -
// a01*a10` leaves `a00*a11 - a01*a10`, the two-by-two determinant.  (The full
// expansion `char_poly(A, lam) = lam^2 - trace(A)*lam + det(A)` is a pure
// commutativity regrouping; see the note at the end of this file.)

/// The characteristic polynomial at zero is the determinant.
theorem char_poly_2x2_zero[R: Ring](n: Nat, a: Matrix[R, n.suc.suc, n.suc.suc]) {
    char_poly_2x2[R](n, a, R.0) = matrix2_det_entries[R](n, a)
} by {
    char_poly_2x2[R](n, a, R.0) =
        (matrix2_00[R](n, a) - R.0) * (matrix2_11[R](n, a) - R.0) - matrix2_01[R](n, a) * matrix2_10[R](n, a)
    sub_zero_right[R](matrix2_00[R](n, a))
    matrix2_00[R](n, a) - R.0 = matrix2_00[R](n, a)
    sub_zero_right[R](matrix2_11[R](n, a))
    matrix2_11[R](n, a) - R.0 = matrix2_11[R](n, a)
    (matrix2_00[R](n, a) - R.0) * (matrix2_11[R](n, a) - R.0) - matrix2_01[R](n, a) * matrix2_10[R](n, a) =
        matrix2_00[R](n, a) * matrix2_11[R](n, a) - matrix2_01[R](n, a) * matrix2_10[R](n, a)
    matrix2_det_entries[R](n, a) =
        matrix2_00[R](n, a) * matrix2_11[R](n, a) - matrix2_01[R](n, a) * matrix2_10[R](n, a)
    char_poly_2x2[R](n, a, R.0) = matrix2_det_entries[R](n, a)
}

// ---------------------------------------------------------------------------
// Notes on the remaining gaps
// ---------------------------------------------------------------------------
//
// 1. The product theorem `det(A*B) = det(A)*det(B)` for general `n`: the row
//    multilinearity (from the #1497 work, ported here) and the alternating
//    property (proved here, both at the list level and for the canonical
//    determinant as `matrix_det_suc_equal_rows`) are the two ingredients that
//    reduce it to the Leibniz expansion
//
//        det(A) = sum over row-index assignments of the signed products,
//        restricted to permutations by alternation.
//
//    The remaining work is the "sums over all functions from rows to columns"
//    machinery noted in the TODO in `fin_matrix_det.ac` (the author's own
//    roadmap); the two-by-two case is proved there as
//    `matrix_det_suc_two_by_two_mul`.
//
// 2. The adjugate identity `A * adj(A) = det(A) * I` for general `n` and the
//    inverse properties and Cramer's rule that follow from it (the two-by-two
//    cases of which live in `fin_matrix_inverse.ac`) need, beyond the
//    alternating property delivered here, the Laplace expansion of the
//    canonical determinant along an arbitrary row,
//
//        det(A) = sum_k a(i,k) * alt(i,k) * det(minor(A, i, k)),
//
//    which requires moving a row to the head of the canonical enumeration and
//    tracking the sign; the canonical-split/reindexing machinery for that is
//    the natural next step.  The inverse uniqueness and the identity
//    `(A*B)^-1 = B^-1*A^-1` are proved here (`invertible_inverse_unique`,
//    `invertible_inverse_of_product`); the criterion `det(A) != 0` iff `A` is
//    invertible needs the adjugate identity for the backward direction and the
//    product theorem for the forward direction.
//
// 3. The characteristic polynomial expansion
//    `char_poly(A, lam) = lam^2 - trace(A)*lam + det(A)` is a pure
//    commutativity regrouping; the constant-term identity
//    `char_poly(A, 0) = det(A)` is proved here as `char_poly_2x2_zero`.


// ---------------------------------------------------------------------------
// Inverse properties: uniqueness and multiplicativity
// ---------------------------------------------------------------------------
//
// With matrix multiplication associative (proved above), an invertible matrix
// has a unique inverse, and the inverse of a product is the product of the
// inverses in reverse order.

/// A square matrix is invertible when it admits a two-sided inverse.
define is_invertible[S: Semiring](n: Nat, a: Matrix[S, n.suc, n.suc]) -> Bool {
    exists(b: Matrix[S, n.suc, n.suc]) {
        square_matrix_mul[S](n.suc, a, b) = matrix_one[S](n.suc) and
            square_matrix_mul[S](n.suc, b, a) = matrix_one[S](n.suc)
    }
}

/// The inverse of an invertible matrix is unique.
theorem invertible_inverse_unique[R: Semiring](
    n: Nat, a: Matrix[R, n.suc, n.suc], b: Matrix[R, n.suc, n.suc], c: Matrix[R, n.suc, n.suc]
) {
    square_matrix_mul[R](n.suc, a, b) = matrix_one[R](n.suc) and
    square_matrix_mul[R](n.suc, b, a) = matrix_one[R](n.suc) and
    square_matrix_mul[R](n.suc, a, c) = matrix_one[R](n.suc) and
    square_matrix_mul[R](n.suc, c, a) = matrix_one[R](n.suc)
        implies b = c
} by {
    if square_matrix_mul[R](n.suc, a, b) = matrix_one[R](n.suc) and
        square_matrix_mul[R](n.suc, b, a) = matrix_one[R](n.suc) and
        square_matrix_mul[R](n.suc, a, c) = matrix_one[R](n.suc) and
        square_matrix_mul[R](n.suc, c, a) = matrix_one[R](n.suc) {
        square_matrix_mul_one_right[R](n.suc, b)
        square_matrix_mul[R](n.suc, b, matrix_one[R](n.suc)) = b
        matrix_mul_square_assoc[R](n, b, a, c)
        square_matrix_mul[R](n.suc, square_matrix_mul[R](n.suc, b, a), c) =
            square_matrix_mul[R](n.suc, b, square_matrix_mul[R](n.suc, a, c))
        square_matrix_mul[R](n.suc, b, a) = matrix_one[R](n.suc)
        square_matrix_mul[R](n.suc, a, c) = matrix_one[R](n.suc)
        square_matrix_mul[R](n.suc, square_matrix_mul[R](n.suc, b, a), c) =
            square_matrix_mul[R](n.suc, matrix_one[R](n.suc), c)
        square_matrix_mul_one_left[R](n.suc, c)
        square_matrix_mul[R](n.suc, matrix_one[R](n.suc), c) = c
        square_matrix_mul[R](n.suc, square_matrix_mul[R](n.suc, b, a), c) = c
        square_matrix_mul[R](n.suc, b, square_matrix_mul[R](n.suc, a, c)) = c
        square_matrix_mul[R](n.suc, b, matrix_one[R](n.suc)) = c
        b = c
    }
}

/// The inverse of a product of invertible matrices is the product of the
/// inverses in reverse order.
theorem invertible_inverse_of_product[R: Semiring](
    n: Nat, a: Matrix[R, n.suc, n.suc], b: Matrix[R, n.suc, n.suc],
    a_inv: Matrix[R, n.suc, n.suc], b_inv: Matrix[R, n.suc, n.suc]
) {
    square_matrix_mul[R](n.suc, a, a_inv) = matrix_one[R](n.suc) and
    square_matrix_mul[R](n.suc, a_inv, a) = matrix_one[R](n.suc) and
    square_matrix_mul[R](n.suc, b, b_inv) = matrix_one[R](n.suc) and
    square_matrix_mul[R](n.suc, b_inv, b) = matrix_one[R](n.suc) implies
        square_matrix_mul[R](n.suc, square_matrix_mul[R](n.suc, a, b),
            square_matrix_mul[R](n.suc, b_inv, a_inv)) = matrix_one[R](n.suc) and
        square_matrix_mul[R](n.suc, square_matrix_mul[R](n.suc, b_inv, a_inv),
            square_matrix_mul[R](n.suc, a, b)) = matrix_one[R](n.suc)
} by {
    if square_matrix_mul[R](n.suc, a, a_inv) = matrix_one[R](n.suc) and
        square_matrix_mul[R](n.suc, a_inv, a) = matrix_one[R](n.suc) and
        square_matrix_mul[R](n.suc, b, b_inv) = matrix_one[R](n.suc) and
        square_matrix_mul[R](n.suc, b_inv, b) = matrix_one[R](n.suc) {
        // (a*b)*(b_inv*a_inv) = a*(b*b_inv)*a_inv = a*one*a_inv = a*a_inv = one
        matrix_mul_square_assoc[R](n, a, b, square_matrix_mul[R](n.suc, b_inv, a_inv))
        square_matrix_mul[R](n.suc, square_matrix_mul[R](n.suc, a, b),
            square_matrix_mul[R](n.suc, b_inv, a_inv)) =
            square_matrix_mul[R](n.suc, a, square_matrix_mul[R](n.suc, b,
                square_matrix_mul[R](n.suc, b_inv, a_inv)))
        matrix_mul_square_assoc[R](n, b, b_inv, a_inv)
        square_matrix_mul[R](n.suc, square_matrix_mul[R](n.suc, b, b_inv), a_inv) =
            square_matrix_mul[R](n.suc, b, square_matrix_mul[R](n.suc, b_inv, a_inv))
        square_matrix_mul[R](n.suc, b, b_inv) = matrix_one[R](n.suc)
        square_matrix_mul_one_left[R](n.suc, a_inv)
        square_matrix_mul[R](n.suc, matrix_one[R](n.suc), a_inv) = a_inv
        square_matrix_mul[R](n.suc, b, square_matrix_mul[R](n.suc, b_inv, a_inv)) = a_inv
        square_matrix_mul[R](n.suc, a, square_matrix_mul[R](n.suc, b,
            square_matrix_mul[R](n.suc, b_inv, a_inv))) =
            square_matrix_mul[R](n.suc, a, a_inv)
        square_matrix_mul[R](n.suc, a, a_inv) = matrix_one[R](n.suc)
        square_matrix_mul[R](n.suc, square_matrix_mul[R](n.suc, a, b),
            square_matrix_mul[R](n.suc, b_inv, a_inv)) = matrix_one[R](n.suc)
        // (b_inv*a_inv)*(a*b) = b_inv*(a_inv*a)*b = b_inv*one*b = b_inv*b = one
        matrix_mul_square_assoc[R](n, b_inv, a_inv, square_matrix_mul[R](n.suc, a, b))
        square_matrix_mul[R](n.suc, square_matrix_mul[R](n.suc, b_inv, a_inv),
            square_matrix_mul[R](n.suc, a, b)) =
            square_matrix_mul[R](n.suc, b_inv, square_matrix_mul[R](n.suc, a_inv,
                square_matrix_mul[R](n.suc, a, b)))
        matrix_mul_square_assoc[R](n, a_inv, a, b)
        square_matrix_mul[R](n.suc, square_matrix_mul[R](n.suc, a_inv, a), b) =
            square_matrix_mul[R](n.suc, a_inv, square_matrix_mul[R](n.suc, a, b))
        square_matrix_mul[R](n.suc, a_inv, a) = matrix_one[R](n.suc)
        square_matrix_mul_one_left[R](n.suc, b)
        square_matrix_mul[R](n.suc, matrix_one[R](n.suc), b) = b
        square_matrix_mul[R](n.suc, a_inv, square_matrix_mul[R](n.suc, a, b)) = b
        square_matrix_mul[R](n.suc, b_inv, square_matrix_mul[R](n.suc, a_inv,
            square_matrix_mul[R](n.suc, a, b))) = square_matrix_mul[R](n.suc, b_inv, b)
        square_matrix_mul[R](n.suc, b_inv, b) = matrix_one[R](n.suc)
        square_matrix_mul[R](n.suc, square_matrix_mul[R](n.suc, b_inv, a_inv),
            square_matrix_mul[R](n.suc, a, b)) = matrix_one[R](n.suc)
        square_matrix_mul[R](n.suc, square_matrix_mul[R](n.suc, a, b),
            square_matrix_mul[R](n.suc, b_inv, a_inv)) = matrix_one[R](n.suc) and
            square_matrix_mul[R](n.suc, square_matrix_mul[R](n.suc, b_inv, a_inv),
                square_matrix_mul[R](n.suc, a, b)) = matrix_one[R](n.suc)
    }
}

/// An invertible matrix and its inverse are each other's inverses.
theorem invertible_inverse_symmetry[R: Semiring](
    n: Nat, a: Matrix[R, n.suc, n.suc], b: Matrix[R, n.suc, n.suc]
) {
    square_matrix_mul[R](n.suc, a, b) = matrix_one[R](n.suc) and
    square_matrix_mul[R](n.suc, b, a) = matrix_one[R](n.suc) implies
        square_matrix_mul[R](n.suc, b, a) = matrix_one[R](n.suc) and
        square_matrix_mul[R](n.suc, a, b) = matrix_one[R](n.suc)
} by {
    if square_matrix_mul[R](n.suc, a, b) = matrix_one[R](n.suc) and
        square_matrix_mul[R](n.suc, b, a) = matrix_one[R](n.suc) {
        square_matrix_mul[R](n.suc, b, a) = matrix_one[R](n.suc) and
            square_matrix_mul[R](n.suc, a, b) = matrix_one[R](n.suc)
    }
}
