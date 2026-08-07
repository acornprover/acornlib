from nat import Nat
from data.fin.fin import Fin, fin_zero
from list import List, map, map_contains, list_max, list_max_contains,
    contains_lte_list_max, map_contains_of_contains
from order import LinearOrder
from data.fin.fin_enum import fin_enum_suc, fin_enum_suc_contains

numerals Nat

/// The list of values a function takes on `Fin[n + 1]`.
///
/// Nonempty by construction, which is what the list maximum needs.
define fin_value_list[T: LinearOrder](n: Nat, f: Fin[n.suc] -> T) -> List[T] {
    map(fin_enum_suc(n), f)
}

/// Every value of the function appears in its value list.
theorem fin_value_list_contains[T: LinearOrder](n: Nat, f: Fin[n.suc] -> T, i: Fin[n.suc]) {
    fin_value_list[T](n, f).contains(f(i))
} by {
    fin_enum_suc_contains(n, i)
    fin_enum_suc(n).contains(i)
    map_contains_of_contains(fin_enum_suc(n), f, i)
    map(fin_enum_suc(n), f).contains(f(i))
    fin_value_list[T](n, f).contains(f(i))
}

/// Every member of the value list is a value of the function.
theorem fin_value_list_witness[T: LinearOrder](n: Nat, f: Fin[n.suc] -> T, y: T) {
    fin_value_list[T](n, f).contains(y) implies exists(i: Fin[n.suc]) { f(i) = y }
} by {
    if fin_value_list[T](n, f).contains(y) {
        map(fin_enum_suc(n), f).contains(y)
        map_contains(fin_enum_suc(n), f, y)
        exists(i: Fin[n.suc]) {
            fin_enum_suc(n).contains(i) and f(i) = y
        }
        let (i: Fin[n.suc]) satisfy {
            fin_enum_suc(n).contains(i) and f(i) = y
        }
        exists(j: Fin[n.suc]) { f(j) = y }
    }
}

/// The largest value a function takes on `Fin[n + 1]`.
///
/// `src/nat_bounded_max.ac` maximises a predicate on the naturals; this maximises a function
/// into any linear order, which is what an argument that picks out a largest coordinate needs.
/// The index type is `Fin[n + 1]` rather than `Fin[n]` because a maximum over an empty index
/// set does not exist.
define fin_function_max[T: LinearOrder](n: Nat, f: Fin[n.suc] -> T) -> T {
    match fin_value_list(n, f) {
        List.nil {
            f(fin_zero(n))
        }
        List.cons(head, tail) {
            list_max(head, tail)
        }
    }
}

/// The value list of a function on `Fin[n + 1]` is a cons.
///
/// It has length `n + 1`, so it is never nil, and the maximum below reads off the cons branch.
theorem fin_value_list_cons[T: LinearOrder](n: Nat, f: Fin[n.suc] -> T) {
    exists(head: T, tail: List[T]) {
        fin_value_list[T](n, f) = List.cons(head, tail)
    }
} by {
    fin_value_list_contains[T](n, f, fin_zero(n))
    fin_value_list[T](n, f).contains(f(fin_zero(n)))
    match fin_value_list[T](n, f) {
        List.nil {
            not List.nil[T].contains(f(fin_zero(n)))
            false
        }
        List.cons(head, tail) {
            exists(h: T, t: List[T]) {
                fin_value_list[T](n, f) = List.cons(h, t)
            }
        }
    }
}

/// No value of the function exceeds the maximum.
theorem fin_function_max_is_upper_bound[T: LinearOrder](
    n: Nat, f: Fin[n.suc] -> T, i: Fin[n.suc]
) {
    f(i) <= fin_function_max[T](n, f)
} by {
    fin_value_list_cons[T](n, f)
    let (head: T, tail: List[T]) satisfy {
        fin_value_list[T](n, f) = List.cons(head, tail)
    }
    fin_function_max[T](n, f) = list_max(head, tail)
    fin_value_list_contains[T](n, f, i)
    fin_value_list[T](n, f).contains(f(i))
    List.cons(head, tail).contains(f(i))
    contains_lte_list_max(head, tail, f(i))
    f(i) <= list_max(head, tail)
    f(i) <= fin_function_max[T](n, f)
}

/// The maximum is attained at some index.
theorem fin_function_max_attained[T: LinearOrder](n: Nat, f: Fin[n.suc] -> T) {
    exists(i: Fin[n.suc]) { f(i) = fin_function_max[T](n, f) }
} by {
    fin_value_list_cons[T](n, f)
    let (head: T, tail: List[T]) satisfy {
        fin_value_list[T](n, f) = List.cons(head, tail)
    }
    fin_function_max[T](n, f) = list_max(head, tail)
    list_max_contains(head, tail)
    List.cons(head, tail).contains(list_max(head, tail))
    fin_value_list[T](n, f).contains(list_max(head, tail))
    fin_value_list[T](n, f).contains(fin_function_max[T](n, f))
    fin_value_list_witness[T](n, f, fin_function_max[T](n, f))
    exists(i: Fin[n.suc]) { f(i) = fin_function_max[T](n, f) }
}

/// A bound on every value bounds the maximum.
theorem fin_function_max_le[T: LinearOrder](n: Nat, f: Fin[n.suc] -> T, b: T) {
    (forall(i: Fin[n.suc]) { f(i) <= b }) implies fin_function_max[T](n, f) <= b
} by {
    if forall(i: Fin[n.suc]) { f(i) <= b } {
        fin_function_max_attained[T](n, f)
        let (i: Fin[n.suc]) satisfy {
            f(i) = fin_function_max[T](n, f)
        }
        f(i) <= b
        fin_function_max[T](n, f) <= b
    }
}
