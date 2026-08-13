from nat import Nat
from data.fin.fin import Fin, fin_zero, fin_succ
from algebra.ring.ring import Ring
from algebra.module.module import ring_as_module
from data.fin.fin_matrix import Matrix, matrix_entry, matrix_one, matrix_smul, square_matrix_mul

numerals Nat

// ---------------------------------------------------------------------------
// Two-by-two entry selectors and the characteristic polynomial
// ---------------------------------------------------------------------------
//
// This module holds only definitions.  Applying a function whose signature
// carries a typeclass-constrained parameter in a matrix type position with a
// *concrete* bound (such as `Nat.0.suc.suc`) to a matrix inside a theorem
// trips a certificate-generation bug (the typeclass parameter is
// re-serialized into a type position), so every selector below keeps the
// bound abstract, as a `n: Nat` parameter over `Matrix[R, n.suc.suc, n.suc.suc]`,
// and is instantiated at `Nat.0` where the two-by-two theorems use it.

/// The upper-left entry of a `(n + 2)` by `(n + 2)` matrix; the two-by-two case is `n = 0`.
define matrix2_00[R](n: Nat, a: Matrix[R, n.suc.suc, n.suc.suc]) -> R {
    matrix_entry[R](n.suc.suc, n.suc.suc, a, fin_zero(n.suc), fin_zero(n.suc))
}

/// The upper-right entry of a `(n + 2)` by `(n + 2)` matrix.
define matrix2_01[R](n: Nat, a: Matrix[R, n.suc.suc, n.suc.suc]) -> R {
    matrix_entry[R](n.suc.suc, n.suc.suc, a, fin_zero(n.suc), fin_succ(n.suc, fin_zero(n)))
}

/// The lower-left entry of a `(n + 2)` by `(n + 2)` matrix.
define matrix2_10[R](n: Nat, a: Matrix[R, n.suc.suc, n.suc.suc]) -> R {
    matrix_entry[R](n.suc.suc, n.suc.suc, a, fin_succ(n.suc, fin_zero(n)), fin_zero(n.suc))
}

/// The lower-right entry of a `(n + 2)` by `(n + 2)` matrix.
define matrix2_11[R](n: Nat, a: Matrix[R, n.suc.suc, n.suc.suc]) -> R {
    matrix_entry[R](n.suc.suc, n.suc.suc, a, fin_succ(n.suc, fin_zero(n)), fin_succ(n.suc, fin_zero(n)))
}

/// The trace of a two-by-two matrix, the sum of its diagonal entries.
define matrix2_trace[R: Ring](n: Nat, a: Matrix[R, n.suc.suc, n.suc.suc]) -> R {
    matrix2_00[R](n, a) + matrix2_11[R](n, a)
}

/// The determinant of a two-by-two matrix, as the entry formula `ad - bc`.
define matrix2_det_entries[R: Ring](n: Nat, a: Matrix[R, n.suc.suc, n.suc.suc]) -> R {
    matrix2_00[R](n, a) * matrix2_11[R](n, a) - matrix2_01[R](n, a) * matrix2_10[R](n, a)
}

/// The characteristic polynomial of a two-by-two matrix, evaluated at `lam`.
///
/// The coefficient form of the determinant of `lam * I - A` (equivalently of
/// `A - lam * I`; the two agree in even dimension), written entrywise rather
/// than through a polynomial-valued matrix: `(a00 - lam) * (a11 - lam) -
/// a01 * a10`.  Expanding gives `lam^2 - trace(A) * lam + det(A)`, which is
/// `char_poly_2x2_standard` below.
define char_poly_2x2[R: Ring](n: Nat, a: Matrix[R, n.suc.suc, n.suc.suc], lam: R) -> R {
    (matrix2_00[R](n, a) - lam) * (matrix2_11[R](n, a) - lam) - matrix2_01[R](n, a) * matrix2_10[R](n, a)
}

/// The characteristic polynomial of a two-by-two matrix in standard form.
///
/// The expanded form `lam^2 - trace(A) * lam + det(A)`.
define char_poly_2x2_standard[R: Ring](n: Nat, a: Matrix[R, n.suc.suc, n.suc.suc], lam: R) -> R {
    lam * lam - matrix2_trace[R](n, a) * lam + matrix2_det_entries[R](n, a)
}

/// The left-hand side of the two-by-two Cayley-Hamilton identity,
/// `A^2 - trace(A) * A + det(A) * I`, with the bound kept abstract.
define cayley_hamilton_lhs[R: Ring](n: Nat, a: Matrix[R, n.suc.suc, n.suc.suc]) -> Matrix[R, n.suc.suc, n.suc.suc] {
    square_matrix_mul[R](n.suc.suc, a, a) -
        matrix_smul[R, R](ring_as_module[R], n.suc.suc, n.suc.suc, matrix2_trace[R](n, a), a) +
        matrix_smul[R, R](ring_as_module[R], n.suc.suc, n.suc.suc, matrix2_det_entries[R](n, a), matrix_one[R](n.suc.suc))
}

/// The coordinate vector `(x0, x1)` on `Fin[2]`, with the bound kept abstract.
///
/// The bound is the index set itself (used at `n = 2`), rather than an offset
/// `n.suc.suc`, because certificate serialization rejects applications of
/// offset-bound functions at concrete bounds.
define matrix2_vec[R](n: Nat, x0: R, x1: R, i: Fin[n]) -> R {
    if i.value = Nat.0 {
        x0
    } else {
        x1
    }
}
