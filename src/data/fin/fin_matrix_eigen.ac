from nat import Nat
from data.fin.fin import Fin, fin_zero
from semiring import Semiring
from comm_ring import CommRing
from algebra.field.field import Field, field_mul_eq_zero
from algebra.add_group import left_cancel
from algebra.ring.ring import mul_sub_left, mul_sub_right
from algebra.ring.ring_axioms_deep import ring_sub_eq_zero_iff_eq
from algebra.module.module import Module, ring_as_module
from data.fin.fin_sum import fin_sum, fin_sum_pointwise_eq, fin_zero_function, fin_sum_zero_function,
    fin_pointwise_add, fin_sum_add
from data.fin.fin_sum_scalar import fin_scale, fin_sum_scalar_mul
from data.fin.fin_matrix import Matrix, matrix_entry
from data.fin.fin_vector import fin_vector_zero, fin_vector_ext, fin_vector_zero_apply,
    fin_vector_add, fin_vector_add_apply, fin_vector_smul, fin_vector_smul_apply

numerals Nat

/// The `k`th term in the entry of a matrix applied to a vector.
define matrix_apply_term[S: Semiring](
    n: Nat, a: Matrix[S, n, n], v: Fin[n] -> S, i: Fin[n], k: Fin[n]
) -> S {
    matrix_entry[S](n, n, a, i, k) * v(k)
}

/// The `i`th entry of a square matrix applied to a vector.
///
/// Vectors are functions on `Fin[n]` rather than one-column matrices, which is the shape
/// `src/fin_vector.ac` already uses and keeps the eigenvalue equation pointwise.
define matrix_apply[S: Semiring](
    n: Nat, a: Matrix[S, n, n], v: Fin[n] -> S, i: Fin[n]
) -> S {
    fin_sum[S](n, matrix_apply_term[S](n, a, v, i))
}

/// The entry of a matrix applied to a vector, written out.
theorem matrix_apply_eq[S: Semiring](
    n: Nat, a: Matrix[S, n, n], v: Fin[n] -> S, i: Fin[n]
) {
    matrix_apply[S](n, a, v, i) = fin_sum[S](n, matrix_apply_term[S](n, a, v, i))
}

/// Applying a matrix to pointwise equal vectors gives pointwise equal results.
theorem matrix_apply_pointwise_eq[S: Semiring](
    n: Nat, a: Matrix[S, n, n], v: Fin[n] -> S, w: Fin[n] -> S, i: Fin[n]
) {
    (forall(k: Fin[n]) { v(k) = w(k) })
        implies matrix_apply[S](n, a, v, i) = matrix_apply[S](n, a, w, i)
} by {
    if forall(k: Fin[n]) { v(k) = w(k) } {
        forall(k: Fin[n]) {
            v(k) = w(k)
            matrix_apply_term[S](n, a, v, i, k) = matrix_entry[S](n, n, a, i, k) * v(k)
            matrix_apply_term[S](n, a, w, i, k) = matrix_entry[S](n, n, a, i, k) * w(k)
            matrix_apply_term[S](n, a, v, i, k) = matrix_apply_term[S](n, a, w, i, k)
        }
        fin_sum_pointwise_eq[S](n, matrix_apply_term[S](n, a, v, i),
            matrix_apply_term[S](n, a, w, i))
        (fin_sum[S](n, matrix_apply_term[S](n, a, v, i))
            = fin_sum[S](n, matrix_apply_term[S](n, a, w, i)))
        matrix_apply[S](n, a, v, i) = matrix_apply[S](n, a, w, i)
    }
}

/// True if some entry of the vector is nonzero.
define is_nonzero_vector[S: Semiring](n: Nat, v: Fin[n] -> S) -> Bool {
    exists(i: Fin[n]) {
        v(i) != S.0
    }
}

/// A nonzero entry witnesses a nonzero vector.
theorem is_nonzero_vector_intro[S: Semiring](n: Nat, v: Fin[n] -> S, i: Fin[n]) {
    v(i) != S.0 implies is_nonzero_vector[S](n, v)
} by {
    if v(i) != S.0 {
        is_nonzero_vector[S](n, v) = exists(k: Fin[n]) {
            v(k) != S.0
        }
        exists(k: Fin[n]) {
            v(k) != S.0
        }
        is_nonzero_vector[S](n, v)
    }
}

/// A nonzero entry can be extracted.
theorem is_nonzero_vector_witness[S: Semiring](n: Nat, v: Fin[n] -> S) {
    is_nonzero_vector[S](n, v) implies exists(i: Fin[n]) { v(i) != S.0 }
} by {
    if is_nonzero_vector[S](n, v) {
        is_nonzero_vector[S](n, v) = exists(k: Fin[n]) {
            v(k) != S.0
        }
        exists(k: Fin[n]) {
            v(k) != S.0
        }
    }
}

/// The zero vector is not nonzero.
theorem zero_vector_not_nonzero[S: Semiring](n: Nat) {
    not is_nonzero_vector[S](n, fin_vector_zero[S](n))
} by {
    if is_nonzero_vector[S](n, fin_vector_zero[S](n)) {
        is_nonzero_vector_witness[S](n, fin_vector_zero[S](n))
        let (i: Fin[n]) satisfy {
            fin_vector_zero[S](n, i) != S.0
        }
        fin_vector_zero_apply[S](n, i)
        fin_vector_zero[S](n, i) = S.0
        false
    }
    not is_nonzero_vector[S](n, fin_vector_zero[S](n))
}

/// True if `v` is a nonzero vector scaled by `lam` under `a`.
///
/// The eigenvector condition, stated pointwise. Requiring the vector to be nonzero is what
/// makes the notion nontrivial: every scalar scales the zero vector.
define is_eigenvector[S: Semiring](
    n: Nat, a: Matrix[S, n, n], lam: S, v: Fin[n] -> S
) -> Bool {
    is_nonzero_vector[S](n, v) and forall(i: Fin[n]) {
        matrix_apply[S](n, a, v, i) = lam * v(i)
    }
}

/// An eigenvector is nonzero.
theorem is_eigenvector_nonzero[S: Semiring](
    n: Nat, a: Matrix[S, n, n], lam: S, v: Fin[n] -> S
) {
    is_eigenvector[S](n, a, lam, v) implies is_nonzero_vector[S](n, v)
} by {
    if is_eigenvector[S](n, a, lam, v) {
        is_nonzero_vector[S](n, v)
    }
}

/// An eigenvector satisfies the eigenvalue equation at each entry.
theorem is_eigenvector_apply[S: Semiring](
    n: Nat, a: Matrix[S, n, n], lam: S, v: Fin[n] -> S, i: Fin[n]
) {
    is_eigenvector[S](n, a, lam, v) implies matrix_apply[S](n, a, v, i) = lam * v(i)
} by {
    if is_eigenvector[S](n, a, lam, v) {
        is_eigenvector[S](n, a, lam, v) = (is_nonzero_vector[S](n, v) and forall(k: Fin[n]) {
            matrix_apply[S](n, a, v, k) = lam * v(k)
        })
        forall(k: Fin[n]) {
            matrix_apply[S](n, a, v, k) = lam * v(k)
        }
        matrix_apply[S](n, a, v, i) = lam * v(i)
    }
}

/// The two conditions give an eigenvector.
theorem is_eigenvector_intro[S: Semiring](
    n: Nat, a: Matrix[S, n, n], lam: S, v: Fin[n] -> S
) {
    is_nonzero_vector[S](n, v) and (forall(i: Fin[n]) {
        matrix_apply[S](n, a, v, i) = lam * v(i)
    }) implies is_eigenvector[S](n, a, lam, v)
} by {
    if is_nonzero_vector[S](n, v) and forall(i: Fin[n]) {
        matrix_apply[S](n, a, v, i) = lam * v(i)
    } {
        is_eigenvector[S](n, a, lam, v) = (is_nonzero_vector[S](n, v) and forall(k: Fin[n]) {
            matrix_apply[S](n, a, v, k) = lam * v(k)
        })
        is_eigenvector[S](n, a, lam, v)
    }
}

/// True if some nonzero vector is scaled by `lam` under `a`.
///
/// The spectrum is defined through eigenvectors rather than as the roots of the
/// characteristic polynomial, so that it is available before that polynomial is. The two
/// agree over a field, but proving so needs the determinant theory.
define is_eigenvalue[S: Semiring](n: Nat, a: Matrix[S, n, n], lam: S) -> Bool {
    exists(v: Fin[n] -> S) {
        is_eigenvector[S](n, a, lam, v)
    }
}

/// An eigenvector witnesses an eigenvalue.
theorem is_eigenvalue_intro[S: Semiring](
    n: Nat, a: Matrix[S, n, n], lam: S, v: Fin[n] -> S
) {
    is_eigenvector[S](n, a, lam, v) implies is_eigenvalue[S](n, a, lam)
} by {
    if is_eigenvector[S](n, a, lam, v) {
        exists(w: Fin[n] -> S) {
            w = v and is_eigenvector[S](n, a, lam, w)
        }
        exists(w: Fin[n] -> S) {
            is_eigenvector[S](n, a, lam, w)
        }
        is_eigenvalue[S](n, a, lam) = exists(w: Fin[n] -> S) {
            is_eigenvector[S](n, a, lam, w)
        }
        is_eigenvalue[S](n, a, lam)
    }
}

/// An eigenvector can be extracted from an eigenvalue.
///
/// This together with the previous theorem is the statement that an eigenvalue admits a
/// nonzero eigenvector and conversely.
theorem is_eigenvalue_witness[S: Semiring](n: Nat, a: Matrix[S, n, n], lam: S) {
    is_eigenvalue[S](n, a, lam) implies exists(v: Fin[n] -> S) {
        is_eigenvector[S](n, a, lam, v)
    }
} by {
    if is_eigenvalue[S](n, a, lam) {
        is_eigenvalue[S](n, a, lam) = exists(w: Fin[n] -> S) {
            is_eigenvector[S](n, a, lam, w)
        }
        exists(w: Fin[n] -> S) {
            is_eigenvector[S](n, a, lam, w)
        }
    }
}

/// The `k`th entry of a row of the matrix.
define matrix_row_fn[S: Semiring](
    n: Nat, a: Matrix[S, n, n], i: Fin[n], k: Fin[n]
) -> S {
    matrix_entry[S](n, n, a, i, k)
}

/// The sum of the entries in a row.
define matrix_row_sum[S: Semiring](n: Nat, a: Matrix[S, n, n], i: Fin[n]) -> S {
    fin_sum[S](n, matrix_row_fn[S](n, a, i))
}

/// True if every row of the matrix sums to one.
///
/// The row-stochastic condition. The nonnegativity that usually accompanies it is a separate
/// condition, stated below, since it needs an order on the entries and this one does not.
define is_row_stochastic[S: Semiring](n: Nat, a: Matrix[S, n, n]) -> Bool {
    forall(i: Fin[n]) {
        matrix_row_sum[S](n, a, i) = S.1
    }
}

/// Each row of such a matrix sums to one.
theorem is_row_stochastic_apply[S: Semiring](
    n: Nat, a: Matrix[S, n, n], i: Fin[n]
) {
    is_row_stochastic[S](n, a) implies matrix_row_sum[S](n, a, i) = S.1
} by {
    if is_row_stochastic[S](n, a) {
        is_row_stochastic[S](n, a) = forall(k: Fin[n]) {
            matrix_row_sum[S](n, a, k) = S.1
        }
        forall(k: Fin[n]) {
            matrix_row_sum[S](n, a, k) = S.1
        }
        matrix_row_sum[S](n, a, i) = S.1
    }
}

/// The pointwise condition on rows is row-stochasticity.
theorem is_row_stochastic_intro[S: Semiring](n: Nat, a: Matrix[S, n, n]) {
    (forall(i: Fin[n]) { matrix_row_sum[S](n, a, i) = S.1 })
        implies is_row_stochastic[S](n, a)
} by {
    if forall(i: Fin[n]) { matrix_row_sum[S](n, a, i) = S.1 } {
        is_row_stochastic[S](n, a) = forall(k: Fin[n]) {
            matrix_row_sum[S](n, a, k) = S.1
        }
        is_row_stochastic[S](n, a)
    }
}

/// The vector that is one in every entry.
define ones_vector[S: Semiring](n: Nat, i: Fin[n]) -> S {
    S.1
}

/// Applying a matrix to the all-ones vector gives the row sums.
///
/// Each term of the sum is an entry times one, so the sum is the row sum itself.
theorem matrix_apply_ones[S: Semiring](n: Nat, a: Matrix[S, n, n], i: Fin[n]) {
    matrix_apply[S](n, a, ones_vector[S](n), i) = matrix_row_sum[S](n, a, i)
} by {
    forall(k: Fin[n]) {
        ones_vector[S](n, k) = S.1
        (matrix_apply_term[S](n, a, ones_vector[S](n), i, k)
            = matrix_entry[S](n, n, a, i, k) * S.1)
        matrix_entry[S](n, n, a, i, k) * S.1 = matrix_entry[S](n, n, a, i, k)
        matrix_row_fn[S](n, a, i, k) = matrix_entry[S](n, n, a, i, k)
        (matrix_apply_term[S](n, a, ones_vector[S](n), i, k) = matrix_row_fn[S](n, a, i, k))
    }
    fin_sum_pointwise_eq[S](n, matrix_apply_term[S](n, a, ones_vector[S](n), i),
        matrix_row_fn[S](n, a, i))
    (fin_sum[S](n, matrix_apply_term[S](n, a, ones_vector[S](n), i))
        = fin_sum[S](n, matrix_row_fn[S](n, a, i)))
    matrix_apply[S](n, a, ones_vector[S](n), i) = matrix_row_sum[S](n, a, i)
}

/// The all-ones vector is nonzero when the size is positive and one is not zero.
///
/// Both hypotheses are needed: there is no index to point at when the size is zero, and in
/// the zero ring every vector is the zero vector.
theorem ones_vector_nonzero[S: Semiring](n: Nat) {
    S.1 != S.0 implies is_nonzero_vector[S](n.suc, ones_vector[S](n.suc))
} by {
    if S.1 != S.0 {
        ones_vector[S](n.suc, fin_zero(n)) = S.1
        ones_vector[S](n.suc)(fin_zero(n)) != S.0
        is_nonzero_vector_intro[S](n.suc, ones_vector[S](n.suc), fin_zero(n))
        is_nonzero_vector[S](n.suc, ones_vector[S](n.suc))
    }
}

/// A row-stochastic matrix has one as an eigenvalue.
///
/// The all-ones vector is the eigenvector: applying the matrix to it gives the row sums,
/// which are all one, and one times one is one.
theorem row_stochastic_has_eigenvalue_one[S: Semiring](
    n: Nat, a: Matrix[S, n.suc, n.suc]
) {
    is_row_stochastic[S](n.suc, a) and S.1 != S.0
        implies is_eigenvalue[S](n.suc, a, S.1)
} by {
    if is_row_stochastic[S](n.suc, a) and S.1 != S.0 {
        ones_vector_nonzero[S](n)
        is_nonzero_vector[S](n.suc, ones_vector[S](n.suc))
        forall(i: Fin[n.suc]) {
            matrix_apply_ones[S](n.suc, a, i)
            matrix_apply[S](n.suc, a, ones_vector[S](n.suc), i) = matrix_row_sum[S](n.suc, a, i)
            is_row_stochastic_apply[S](n.suc, a, i)
            matrix_row_sum[S](n.suc, a, i) = S.1
            ones_vector[S](n.suc, i) = S.1
            S.1 * S.1 = S.1
            (matrix_apply[S](n.suc, a, ones_vector[S](n.suc), i)
                = S.1 * ones_vector[S](n.suc)(i))
        }
        is_eigenvector_intro[S](n.suc, a, S.1, ones_vector[S](n.suc))
        is_eigenvector[S](n.suc, a, S.1, ones_vector[S](n.suc))
        is_eigenvalue_intro[S](n.suc, a, S.1, ones_vector[S](n.suc))
        is_eigenvalue[S](n.suc, a, S.1)
    }
}

/// Applying a matrix to the zero vector gives the zero vector at every coordinate.
theorem matrix_apply_zero[S: Semiring](n: Nat, a: Matrix[S, n, n], i: Fin[n]) {
    matrix_apply[S](n, a, fin_vector_zero[S](n), i) = S.0
} by {
    matrix_apply_eq[S](n, a, fin_vector_zero[S](n), i)
    forall(k: Fin[n]) {
        matrix_apply_term[S](n, a, fin_vector_zero[S](n), i, k) = matrix_entry[S](n, n, a, i, k) * fin_vector_zero[S](n, k)
        fin_vector_zero_apply[S](n, k)
        matrix_apply_term[S](n, a, fin_vector_zero[S](n), i, k) = matrix_entry[S](n, n, a, i, k) * S.0
        matrix_entry[S](n, n, a, i, k) * S.0 = S.0
        matrix_apply_term[S](n, a, fin_vector_zero[S](n), i, k) = fin_zero_function[S](n, k)
    }
    fin_sum_pointwise_eq[S](n, matrix_apply_term[S](n, a, fin_vector_zero[S](n), i), fin_zero_function[S](n))
    fin_sum[S](n, matrix_apply_term[S](n, a, fin_vector_zero[S](n), i)) = fin_sum[S](n, fin_zero_function[S](n))
    fin_sum_zero_function[S](n)
    matrix_apply[S](n, a, fin_vector_zero[S](n), i) = S.0
}

// ---------------------------------------------------------------------------
// Linearity of the matrix action and independence of eigenvectors
// ---------------------------------------------------------------------------

/// Applying a matrix to a pointwise sum of vectors gives the sum of the applications.
///
/// This is the additive half of the linearity of the matrix action, stated over a
/// commutative ring so that the entrywise products can be re-associated.
theorem matrix_apply_add[R: CommRing](
    n: Nat, a: Matrix[R, n, n], v: Fin[n] -> R, w: Fin[n] -> R, i: Fin[n]
) {
    matrix_apply[R](n, a, fin_vector_add[R](n, v, w), i) =
        matrix_apply[R](n, a, v, i) + matrix_apply[R](n, a, w, i)
} by {
    matrix_apply_eq[R](n, a, fin_vector_add[R](n, v, w), i)
    matrix_apply_eq[R](n, a, v, i)
    matrix_apply_eq[R](n, a, w, i)
    forall(k: Fin[n]) {
        matrix_apply_term[R](n, a, fin_vector_add[R](n, v, w), i, k) =
            matrix_entry[R](n, n, a, i, k) * fin_vector_add[R](n, v, w, k)
        fin_vector_add_apply[R](n, v, w, k)
        matrix_apply_term[R](n, a, fin_vector_add[R](n, v, w), i, k) =
            matrix_entry[R](n, n, a, i, k) * (v(k) + w(k))
        matrix_entry[R](n, n, a, i, k) * (v(k) + w(k)) =
            matrix_entry[R](n, n, a, i, k) * v(k) + matrix_entry[R](n, n, a, i, k) * w(k)
        matrix_apply_term[R](n, a, v, i, k) = matrix_entry[R](n, n, a, i, k) * v(k)
        matrix_apply_term[R](n, a, w, i, k) = matrix_entry[R](n, n, a, i, k) * w(k)
        fin_pointwise_add[R](n, matrix_apply_term[R](n, a, v, i), matrix_apply_term[R](n, a, w, i), k) =
            matrix_apply_term[R](n, a, v, i, k) + matrix_apply_term[R](n, a, w, i, k)
        matrix_apply_term[R](n, a, fin_vector_add[R](n, v, w), i, k) =
            fin_pointwise_add[R](n, matrix_apply_term[R](n, a, v, i), matrix_apply_term[R](n, a, w, i), k)
    }
    fin_sum_pointwise_eq[R](n, matrix_apply_term[R](n, a, fin_vector_add[R](n, v, w), i),
        fin_pointwise_add[R](n, matrix_apply_term[R](n, a, v, i), matrix_apply_term[R](n, a, w, i)))
    fin_sum_add[R](n, matrix_apply_term[R](n, a, v, i), matrix_apply_term[R](n, a, w, i))
    matrix_apply[R](n, a, fin_vector_add[R](n, v, w), i) =
        matrix_apply[R](n, a, v, i) + matrix_apply[R](n, a, w, i)
}

/// Applying a matrix to a scalar multiple of a vector gives the scalar multiple of the application.
///
/// The multiplicative half of linearity, over the ring viewed as a module over itself.
/// Commutativity of the ring is what lets the scalar pass through the entry product.
theorem matrix_apply_smul[R: CommRing](
    n: Nat, a: Matrix[R, n, n], c: R, v: Fin[n] -> R, i: Fin[n]
) {
    matrix_apply[R](n, a, fin_vector_smul[R, R](ring_as_module[R], n, c, v), i) =
        c * matrix_apply[R](n, a, v, i)
} by {
    matrix_apply_eq[R](n, a, fin_vector_smul[R, R](ring_as_module[R], n, c, v), i)
    matrix_apply_eq[R](n, a, v, i)
    forall(k: Fin[n]) {
        matrix_apply_term[R](n, a, fin_vector_smul[R, R](ring_as_module[R], n, c, v), i, k) =
            matrix_entry[R](n, n, a, i, k) * fin_vector_smul[R, R](ring_as_module[R], n, c, v, k)
        fin_vector_smul_apply[R, R](ring_as_module[R], n, c, v, k)
        matrix_apply_term[R](n, a, fin_vector_smul[R, R](ring_as_module[R], n, c, v), i, k) =
            matrix_entry[R](n, n, a, i, k) * (c * v(k))
        matrix_entry[R](n, n, a, i, k) * (c * v(k)) =
            c * (matrix_entry[R](n, n, a, i, k) * v(k))
        matrix_apply_term[R](n, a, v, i, k) = matrix_entry[R](n, n, a, i, k) * v(k)
        fin_scale[R](n, c, matrix_apply_term[R](n, a, v, i), k) =
            c * matrix_apply_term[R](n, a, v, i, k)
        matrix_apply_term[R](n, a, fin_vector_smul[R, R](ring_as_module[R], n, c, v), i, k) =
            fin_scale[R](n, c, matrix_apply_term[R](n, a, v, i), k)
    }
    fin_sum_pointwise_eq[R](n, matrix_apply_term[R](n, a, fin_vector_smul[R, R](ring_as_module[R], n, c, v), i),
        fin_scale[R](n, c, matrix_apply_term[R](n, a, v, i)))
    fin_sum_scalar_mul[R](n, c, matrix_apply_term[R](n, a, v, i))
    matrix_apply[R](n, a, fin_vector_smul[R, R](ring_as_module[R], n, c, v), i) =
        c * matrix_apply[R](n, a, v, i)
}

/// The pointwise linear combination `s * v + t * w` of two coordinate vectors.
define fin_vector_lin_comb2[S: Semiring](
    n: Nat, s: S, v: Fin[n] -> S, t: S, w: Fin[n] -> S, i: Fin[n]
) -> S {
    s * v(i) + t * w(i)
}

/// Applying a matrix to a linear combination of two vectors gives the linear combination
/// of the applications.
///
/// The two linearity halves assembled into the form the independence argument needs.
theorem matrix_apply_lin_comb2[R: CommRing](
    n: Nat, a: Matrix[R, n, n], s: R, v: Fin[n] -> R, t: R, w: Fin[n] -> R, i: Fin[n]
) {
    matrix_apply[R](n, a, fin_vector_lin_comb2[R](n, s, v, t, w), i) =
        s * matrix_apply[R](n, a, v, i) + t * matrix_apply[R](n, a, w, i)
} by {
    let sv = fin_vector_smul[R, R](ring_as_module[R], n, s, v)
    let tw = fin_vector_smul[R, R](ring_as_module[R], n, t, w)
    forall(k: Fin[n]) {
        fin_vector_lin_comb2[R](n, s, v, t, w, k) = s * v(k) + t * w(k)
        fin_vector_smul_apply[R, R](ring_as_module[R], n, s, v, k)
        fin_vector_smul_apply[R, R](ring_as_module[R], n, t, w, k)
        sv(k) = s * v(k)
        tw(k) = t * w(k)
        fin_vector_add_apply[R](n, sv, tw, k)
        fin_vector_add[R](n, sv, tw, k) = sv(k) + tw(k)
        fin_vector_lin_comb2[R](n, s, v, t, w, k) = fin_vector_add[R](n, sv, tw, k)
    }
    matrix_apply_pointwise_eq[R](n, a, fin_vector_lin_comb2[R](n, s, v, t, w), fin_vector_add[R](n, sv, tw), i)
    matrix_apply_add[R](n, a, sv, tw, i)
    matrix_apply_smul[R](n, a, s, v, i)
    matrix_apply_smul[R](n, a, t, w, i)
    matrix_apply[R](n, a, fin_vector_lin_comb2[R](n, s, v, t, w), i) =
        s * matrix_apply[R](n, a, v, i) + t * matrix_apply[R](n, a, w, i)
}

/// Two zero linear equations with a common first summand give the factored difference
/// of the remaining summands.
///
/// This is the algebraic core of the Vandermonde argument: subtracting the equations
/// annihilates the shared term and leaves the difference of the others, factored.
theorem eigen_combine_equations[R: CommRing](s: R, t: R, lam: R, mu: R, v: R, w: R) {
    s * lam * v + t * mu * w = R.0 and s * lam * v + t * lam * w = R.0
        implies t * (mu - lam) * w = R.0
} by {
    if s * lam * v + t * mu * w = R.0 and s * lam * v + t * lam * w = R.0 {
        left_cancel(s * lam * v, t * mu * w, t * lam * w)
        t * mu * w = t * lam * w
        t * mu * w - t * lam * w = R.0
        mul_sub_left[R](t, mu, lam)
        t * (mu - lam) = t * mu - t * lam
        (t * (mu - lam)) * w = (t * mu - t * lam) * w
        mul_sub_right[R](t * mu, t * lam, w)
        (t * mu - t * lam) * w = t * mu * w - t * lam * w
        t * (mu - lam) * w = R.0
    }
}

/// Eigenvectors for distinct eigenvalues are linearly independent.
///
/// Two eigenvectors of a matrix for distinct eigenvalues over a field are linearly
/// independent: the only scalar combination that is the zero vector pointwise is the
/// trivial one. This is the two-vector case of the classical Vandermonde argument:
/// applying the matrix to the combination and subtracting the eigenvalue multiple of
/// the combination kills the first vector and leaves `t * (mu - lam) * w = 0`.
theorem eigenvectors_distinct_eigenvalues_independent[F: Field](
    n: Nat, a: Matrix[F, n, n], lam: F, mu: F, v: Fin[n] -> F, w: Fin[n] -> F
) {
    lam != mu and is_eigenvector[F](n, a, lam, v) and is_eigenvector[F](n, a, mu, w)
        implies forall(s: F, t: F) {
            (forall(i: Fin[n]) { fin_vector_lin_comb2[F](n, s, v, t, w, i) = F.0 })
                implies s = F.0 and t = F.0
        }
} by {
    if lam != mu and is_eigenvector[F](n, a, lam, v) and is_eigenvector[F](n, a, mu, w) {
        is_eigenvector_nonzero[F](n, a, lam, v)
        is_eigenvector_nonzero[F](n, a, mu, w)
        is_nonzero_vector_witness[F](n, v)
        is_nonzero_vector_witness[F](n, w)
        let (k: Fin[n]) satisfy { v(k) != F.0 }
        let (j: Fin[n]) satisfy { w(j) != F.0 }
        forall(s: F, t: F) {
            if forall(i: Fin[n]) { fin_vector_lin_comb2[F](n, s, v, t, w, i) = F.0 } {
                // The combination is the zero vector, so the matrix sends it to zero.
                forall(i: Fin[n]) {
                    fin_vector_lin_comb2[F](n, s, v, t, w, i) = fin_vector_zero[F](n, i)
                }
                matrix_apply_pointwise_eq[F](n, a, fin_vector_lin_comb2[F](n, s, v, t, w),
                    fin_vector_zero[F](n), j)
                matrix_apply_zero[F](n, a, j)
                matrix_apply[F](n, a, fin_vector_lin_comb2[F](n, s, v, t, w), j) = F.0
                // Linearity expresses the same value as the combination of the actions.
                matrix_apply_lin_comb2[F](n, a, s, v, t, w, j)
                matrix_apply[F](n, a, fin_vector_lin_comb2[F](n, s, v, t, w), j) =
                    s * matrix_apply[F](n, a, v, j) + t * matrix_apply[F](n, a, w, j)
                is_eigenvector_apply[F](n, a, lam, v, j)
                is_eigenvector_apply[F](n, a, mu, w, j)
                matrix_apply[F](n, a, v, j) = lam * v(j)
                matrix_apply[F](n, a, w, j) = mu * w(j)
                s * (lam * v(j)) + t * (mu * w(j)) = F.0
                s * lam * v(j) + t * mu * w(j) = F.0
                // The eigenvalue multiple of the combination is also zero.
                fin_vector_lin_comb2[F](n, s, v, t, w, j) = s * v(j) + t * w(j)
                s * v(j) + t * w(j) = F.0
                lam * (s * v(j) + t * w(j)) = lam * F.0
                lam * F.0 = F.0
                lam * (s * v(j) + t * w(j)) = F.0
                lam * s * v(j) + lam * t * w(j) = F.0
                s * lam * v(j) + t * lam * w(j) = F.0
                // Subtracting the two equations leaves t * (mu - lam) * w(j) = 0.
                eigen_combine_equations[F](s, t, lam, mu, v(j), w(j))
                t * (mu - lam) * w(j) = F.0
                // Over a field, with mu != lam and w(j) != 0, the coefficient t vanishes.
                field_mul_eq_zero[F](t, (mu - lam) * w(j))
                t = F.0 or (mu - lam) * w(j) = F.0
                if t = F.0 {
                    t = F.0
                } else {
                    (mu - lam) * w(j) = F.0
                    field_mul_eq_zero[F](mu - lam, w(j))
                    mu - lam = F.0 or w(j) = F.0
                    if mu - lam = F.0 {
                        ring_sub_eq_zero_iff_eq[F](mu, lam)
                        mu = lam
                        false
                    } else {
                        w(j) = F.0
                        false
                    }
                    false
                }
                t = F.0
                // With t = 0 the original combination at k is s * v(k), which vanishes.
                fin_vector_lin_comb2[F](n, s, v, t, w, k) = s * v(k) + t * w(k)
                s * v(k) + t * w(k) = F.0
                t * w(k) = F.0
                s * v(k) + F.0 = s * v(k)
                s * v(k) = F.0
                field_mul_eq_zero[F](s, v(k))
                s = F.0 or v(k) = F.0
                if s = F.0 {
                    s = F.0
                } else {
                    v(k) = F.0
                    false
                }
                s = F.0
                s = F.0 and t = F.0
            }
        }
    }
}

/// Eigenvectors for distinct eigenvalues are independent, for two fixed coefficients.
///
/// The parameterized form of `eigenvectors_distinct_eigenvalues_independent`: the
/// argument is the same, but the coefficients are theorem parameters rather than
/// quantified, which is the shape theorem citation can instantiate directly.
theorem eigenvectors_independent_at[F: Field](
    n: Nat, a: Matrix[F, n, n], lam: F, mu: F, v: Fin[n] -> F, w: Fin[n] -> F, s: F, t: F
) {
    lam != mu and is_eigenvector[F](n, a, lam, v) and is_eigenvector[F](n, a, mu, w) and
    (forall(i: Fin[n]) { fin_vector_lin_comb2[F](n, s, v, t, w, i) = F.0 })
        implies s = F.0 and t = F.0
} by {
    if lam != mu and is_eigenvector[F](n, a, lam, v) and is_eigenvector[F](n, a, mu, w) and
        (forall(i: Fin[n]) { fin_vector_lin_comb2[F](n, s, v, t, w, i) = F.0 }) {
        is_eigenvector_nonzero[F](n, a, lam, v)
        is_eigenvector_nonzero[F](n, a, mu, w)
        is_nonzero_vector_witness[F](n, v)
        is_nonzero_vector_witness[F](n, w)
        let (k: Fin[n]) satisfy { v(k) != F.0 }
        let (j: Fin[n]) satisfy { w(j) != F.0 }
        // The combination is the zero vector, so the matrix sends it to zero.
        forall(i: Fin[n]) {
            fin_vector_lin_comb2[F](n, s, v, t, w, i) = fin_vector_zero[F](n, i)
        }
        matrix_apply_pointwise_eq[F](n, a, fin_vector_lin_comb2[F](n, s, v, t, w),
            fin_vector_zero[F](n), j)
        matrix_apply_zero[F](n, a, j)
        matrix_apply[F](n, a, fin_vector_lin_comb2[F](n, s, v, t, w), j) = F.0
        // Linearity expresses the same value as the combination of the actions.
        matrix_apply_lin_comb2[F](n, a, s, v, t, w, j)
        matrix_apply[F](n, a, fin_vector_lin_comb2[F](n, s, v, t, w), j) =
            s * matrix_apply[F](n, a, v, j) + t * matrix_apply[F](n, a, w, j)
        is_eigenvector_apply[F](n, a, lam, v, j)
        is_eigenvector_apply[F](n, a, mu, w, j)
        matrix_apply[F](n, a, v, j) = lam * v(j)
        matrix_apply[F](n, a, w, j) = mu * w(j)
        s * (lam * v(j)) + t * (mu * w(j)) = F.0
        s * lam * v(j) + t * mu * w(j) = F.0
        // The eigenvalue multiple of the combination is also zero.
        fin_vector_lin_comb2[F](n, s, v, t, w, j) = s * v(j) + t * w(j)
        s * v(j) + t * w(j) = F.0
        lam * (s * v(j) + t * w(j)) = lam * F.0
        lam * F.0 = F.0
        lam * (s * v(j) + t * w(j)) = F.0
        lam * s * v(j) + lam * t * w(j) = F.0
        s * lam * v(j) + t * lam * w(j) = F.0
        // Subtracting the two equations leaves t * (mu - lam) * w(j) = 0.
        eigen_combine_equations[F](s, t, lam, mu, v(j), w(j))
        t * (mu - lam) * w(j) = F.0
        // Over a field, with mu != lam and w(j) != 0, the coefficient t vanishes.
        field_mul_eq_zero[F](t, (mu - lam) * w(j))
        t = F.0 or (mu - lam) * w(j) = F.0
        if t = F.0 {
            t = F.0
        } else {
            (mu - lam) * w(j) = F.0
            field_mul_eq_zero[F](mu - lam, w(j))
            mu - lam = F.0 or w(j) = F.0
            if mu - lam = F.0 {
                ring_sub_eq_zero_iff_eq[F](mu, lam)
                mu = lam
                false
            } else {
                w(j) = F.0
                false
            }
            false
        }
        t = F.0
        // With t = 0 the original combination at k is s * v(k), which vanishes.
        fin_vector_lin_comb2[F](n, s, v, t, w, k) = s * v(k) + t * w(k)
        s * v(k) + t * w(k) = F.0
        t * w(k) = F.0
        s * v(k) + F.0 = s * v(k)
        s * v(k) = F.0
        field_mul_eq_zero[F](s, v(k))
        s = F.0 or v(k) = F.0
        if s = F.0 {
            s = F.0
        } else {
            v(k) = F.0
            false
        }
        s = F.0
        s = F.0 and t = F.0
    }
}
