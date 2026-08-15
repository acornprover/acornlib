/// Characteristic polynomials, the Cayley-Hamilton theorem for two-by-two
/// matrices, and the trace.
///
/// The characteristic polynomial is first defined honestly as the determinant
/// of `A - lam * I` (via the canonical determinant on positive-dimensional
/// matrices), then related to the entrywise forms `char_poly_2x2` and
/// `char_poly_2x2_standard` from `fin_matrix_2x2.ac`.  The two-by-two
/// Cayley-Hamilton identity `A^2 - trace(A) * A + det(A) * I = 0` is proved by
/// expansion; it is stated over a commutative ring because the entrywise
/// identity needs `a01 * a10 = a10 * a01`-style commutations.  The
/// characteristic-polynomial characterization of eigenvalues (`lam` is an
/// eigenvalue of `A` iff `det(A - lam * I) = 0` iff `A - lam * I` is not
/// invertible) is proved over a field for two-by-two matrices, reusing the
/// invertibility criterion from `fin_matrix_inverse.ac`.  The final section
/// develops the trace of a square matrix: its basic properties and the
/// identity `trace(A B) = trace(B A)`.
///
/// Following the convention of `fin_matrix_2x2.ac`, every helper below that is
/// applied inside a theorem keeps its size bound abstract (an ambient `n: Nat`
/// parameter over `Matrix[R, n.suc.suc, n.suc.suc]`) and is instantiated at
/// `Nat.0` in the two-by-two theorems, to avoid a certificate-serialization
/// bug with concrete matrix bounds in typeclass parameter positions.  The
/// two-by-two theorems of `fin_matrix_inverse.ac` are stated at the
/// `Nat.0.suc.suc` representation of `Fin[2]`; the two-by-two determinant
/// expansion of `fin_matrix_det.ac` is stated at the `Nat.1.suc`
/// representation, so the bridge between them is re-derived here at the
/// `Nat.0.suc.suc` representation (the two representations are not
/// definitionally equal).

from nat import Nat, lt_suc, lt_trans
from semiring import Semiring
from comm_ring import CommRing
from algebra.field.field import Field, field_mul_eq_zero
from algebra.add_group import left_cancel
from algebra.ring.ring import Ring, mul_sub_left, mul_sub_right, mul_neg_left, mul_neg_right,
    mul_zero_left, mul_zero_right
from algebra.ring.ring_axioms_deep import ring_sub_eq_zero_iff_eq, ring_neg_eq_zero_iff
from algebra.module.module import Module, ring_as_module, ring_as_module_smul
from algebra.add_comm_monoid import AddCommMonoid
from algebra.add_comm_monoid_rearrange import add_swap_inner
from data.fin.fin import Fin, fin_zero, fin_succ, fin_last, fin_zero_value, fin_last_value,
    fin_succ_value, fin_of_nat_suc, fin_of_nat_suc_new_of_lt, fin_zero_new, fin_last_new,
    ext, value_lt_bound
from data.fin.fin_enum import fin_enum_suc, fin_enum_suc_contains
from data.fin.fin_sum_enum import fin_sum_suc_eq_sum_fin_enum_suc
from data.fin.fin_sum import fin_sum, fin_pointwise_add, fin_sum_pointwise_eq, fin_sum_add
from data.fin.fin_sum_scalar import fin_scale, fin_sum_scalar_mul
from data.fin.fin_vector import fin_vector_zero, fin_vector_zero_apply, fin_vector_smul,
    fin_vector_smul_apply
from data.fin.fin_matrix import Matrix, matrix_entry, matrix_ext, matrix_one, matrix_zero,
    matrix_sub, matrix_add, matrix_smul, matrix_transpose, square_matrix_mul,
    square_matrix_mul_add_left, square_matrix_mul_add_right, matrix_entry_add,
    matrix_entry_sub, matrix_entry_smul, matrix_entry_transpose, matrix_entry_one_diag,
    matrix_entry_one_off_diag, matrix_entry_zero, matrix_entry_mul, matrix_mul_term,
    determinant_list, matrix_det_list, square_matrix_entry_function
from data.fin.fin_matrix_2x2 import matrix2_00, matrix2_01, matrix2_10, matrix2_11,
    matrix2_trace, matrix2_det_entries, char_poly_2x2, char_poly_2x2_standard,
    cayley_hamilton_lhs, matrix2_vec
from data.fin.fin_matrix_det import matrix_det_suc, matrix_det_suc_unfold, two_row_list,
    two_col_list, determinant_list_two_two, sub_zero_right, sub_neg_rev, sub_factor,
    sum_map_sum_exchange
from data.fin.fin_matrix_inverse import matrix2_mul_entry_00, matrix2_mul_entry_01,
    matrix2_mul_entry_10, matrix2_mul_entry_11, matrix2_det_unfold,
    matrix2_det_nonzero_implies_invertible, matrix2_invertible_iff_det_nonzero,
    fin_two_index_cases, fin_two_zero_value_two,
    fin_two_one_value_two, matrix2_apply_entry,
    matrix2_apply_one, ring_sub_mul_sub, ring4_sub_swap, ring_sub_add_shift,
    ring_sub_cancel_common
from data.fin.fin_matrix_eigen import is_eigenvalue, is_eigenvector, is_nonzero_vector,
    is_nonzero_vector_intro, is_nonzero_vector_witness, is_eigenvector_apply,
    is_eigenvalue_intro, is_eigenvalue_witness, is_eigenvector_nonzero,
    is_eigenvector_intro, matrix_apply, matrix_apply_eq, matrix_apply_term,
    matrix_apply_pointwise_eq, matrix_apply_zero, matrix_apply_smul
from algebra.matrix_algebra import fin_sum_two, matrix_apply_mul_two, matrix_apply_vec,
    matrix2_det, matrix2_entry_00, matrix2_entry_01, matrix2_entry_10, matrix2_entry_11
from list import List, map, map_singleton, sum, sum_map_of_pointwise

numerals Nat

// ---------------------------------------------------------------------------
// The characteristic polynomial as a determinant
// ---------------------------------------------------------------------------

/// The `A - lam * I` matrix whose determinant is the characteristic polynomial.
define char_poly_matrix[R: Ring](n: Nat, a: Matrix[R, n.suc.suc, n.suc.suc], lam: R) -> Matrix[R, n.suc.suc, n.suc.suc] {
    matrix_sub[R](n.suc.suc, n.suc.suc, a,
        matrix_smul[R, R](ring_as_module[R], n.suc.suc, n.suc.suc, lam, matrix_one[R](n.suc.suc)))
}

/// The characteristic polynomial of a two-by-two matrix as the determinant of
/// `A - lam * I`, at ambient size `n`.
define char_poly_det[R: Ring](n: Nat, a: Matrix[R, n.suc.suc, n.suc.suc], lam: R) -> R {
    matrix_det_suc[R](n.suc, char_poly_matrix[R](n, a, lam))
}

/// The characteristic polynomial unfolds to the determinant of `A - lam * I`.
theorem char_poly_det_unfold[R: Ring](n: Nat, a: Matrix[R, n.suc.suc, n.suc.suc], lam: R) {
    char_poly_det[R](n, a, lam) = matrix_det_suc[R](n.suc, char_poly_matrix[R](n, a, lam))
}

/// The last index of `Fin[2]` is the successor of the first.
theorem fin_last_two_eq_succ {
    fin_last(Nat.0.suc) = fin_succ(Nat.0.suc, fin_zero(Nat.0))
} by {
    fin_last_value(Nat.0.suc)
    fin_last(Nat.0.suc).value = Nat.0.suc
    fin_succ_value(Nat.0.suc, fin_zero(Nat.0))
    fin_succ(Nat.0.suc, fin_zero(Nat.0)).value = Nat.0.suc
    fin_last(Nat.0.suc).value = fin_succ(Nat.0.suc, fin_zero(Nat.0)).value
    ext(Nat.0.suc.suc, fin_last(Nat.0.suc), fin_succ(Nat.0.suc, fin_zero(Nat.0)))
    fin_last(Nat.0.suc) = fin_succ(Nat.0.suc, fin_zero(Nat.0))
}

/// The canonical enumeration of `Fin[2]` is the two successor indices in order.
theorem fin_enum_suc_two {
    fin_enum_suc(Nat.0.suc) =
        List.cons(fin_zero(Nat.0.suc), List.cons(fin_succ(Nat.0.suc, fin_zero(Nat.0)), List.nil[Fin[Nat.0.suc.suc]]))
} by {
    let n = Nat.0.suc
    let z = Nat.0
    z < z.suc
    n.suc.range = n.range.append(n)
    n.range = z.range.append(z)
    z.range = List.nil[Nat]
    z.range.append(z) = List.singleton(z)
    n.range = List.singleton(z)
    n.suc.range = List.singleton(z).append(n)
    List.singleton(z).append(n) = List.singleton(z) + List.singleton(n)
    List.singleton(z) + List.singleton(n) = List.cons(z, List.nil[Nat]) + List.singleton(n)
    List.cons(z, List.nil[Nat]) + List.singleton(n) = List.cons(z, List.nil[Nat] + List.singleton(n))
    List.nil[Nat] + List.singleton(n) = List.singleton(n)
    List.cons(z, List.nil[Nat] + List.singleton(n)) = List.cons(z, List.singleton(n))
    List.singleton(z) + List.singleton(n) = List.cons(z, List.singleton(n))
    List.singleton(z).append(n) = List.cons(z, List.singleton(n))
    n.suc.range = List.cons(z, List.singleton(n))
    fin_enum_suc(n) = map[Nat, Fin[n.suc]](n.suc.range, fin_of_nat_suc(n))
    fin_enum_suc(n) = map[Nat, Fin[n.suc]](List.cons(z, List.singleton(n)), fin_of_nat_suc(n))
    map[Nat, Fin[n.suc]](List.cons(z, List.singleton(n)), fin_of_nat_suc(n)) =
        List.cons(fin_of_nat_suc(n, z), map[Nat, Fin[n.suc]](List.singleton(n), fin_of_nat_suc(n)))
    map_singleton[Nat, Fin[n.suc]](fin_of_nat_suc(n), n)
    map[Nat, Fin[n.suc]](List.singleton(n), fin_of_nat_suc(n)) = List.singleton(fin_of_nat_suc(n, n))
    map[Nat, Fin[n.suc]](List.cons(z, List.singleton(n)), fin_of_nat_suc(n)) =
        List.cons(fin_of_nat_suc(n, z), List.singleton(fin_of_nat_suc(n, n)))
    z < n.suc
    fin_of_nat_suc_new_of_lt(n, z)
    fin_zero_new(n)
    Option.some(fin_of_nat_suc(n, z)) = Option.some(fin_zero(n))
    some_injective[Fin[n.suc]](fin_of_nat_suc(n, z), fin_zero(n))
    fin_of_nat_suc(n, z) = fin_zero(n)
    n < n.suc
    fin_of_nat_suc_new_of_lt(n, n)
    fin_last_new(n)
    Option.some(fin_of_nat_suc(n, n)) = Option.some(fin_last(n))
    some_injective[Fin[n.suc]](fin_of_nat_suc(n, n), fin_last(n))
    fin_of_nat_suc(n, n) = fin_last(n)
    fin_enum_suc(n) = List.cons(fin_zero(n), List.singleton(fin_last(n)))
    fin_last_two_eq_succ
    fin_enum_suc(Nat.0.suc) =
        List.cons(fin_zero(Nat.0.suc), List.singleton(fin_succ(Nat.0.suc, fin_zero(Nat.0))))
    fin_enum_suc(Nat.0.suc) =
        List.cons(fin_zero(Nat.0.suc), List.cons(fin_succ(Nat.0.suc, fin_zero(Nat.0)), List.nil[Fin[Nat.0.suc.suc]]))
}

/// The canonical determinant of a two-by-two matrix unfolds to the
/// determinant-list over the two-element row and column lists.
theorem matrix_det_suc_two_bridge[R: Ring](a: Matrix[R, Nat.0.suc.suc, Nat.0.suc.suc]) {
    matrix_det_suc[R](Nat.0.suc, a) =
        determinant_list[R, Fin[Nat.0.suc.suc]](square_matrix_entry_function[R](Nat.0.suc.suc, a),
            two_row_list[Fin[Nat.0.suc.suc]](fin_zero(Nat.0.suc), fin_succ(Nat.0.suc, fin_zero(Nat.0))),
            two_col_list[Fin[Nat.0.suc.suc]](fin_zero(Nat.0.suc), fin_succ(Nat.0.suc, fin_zero(Nat.0))))
} by {
    matrix_det_suc_unfold[R](Nat.0.suc, a)
    matrix_det_suc[R](Nat.0.suc, a) =
        matrix_det_list[R](Nat.0.suc.suc, a, fin_enum_suc(Nat.0.suc), fin_enum_suc(Nat.0.suc))
    fin_enum_suc_two
    matrix_det_list[R](Nat.0.suc.suc, a, fin_enum_suc(Nat.0.suc), fin_enum_suc(Nat.0.suc)) =
        matrix_det_list[R](Nat.0.suc.suc, a,
            List.cons(fin_zero(Nat.0.suc), List.cons(fin_succ(Nat.0.suc, fin_zero(Nat.0)), List.nil[Fin[Nat.0.suc.suc]])),
            List.cons(fin_zero(Nat.0.suc), List.cons(fin_succ(Nat.0.suc, fin_zero(Nat.0)), List.nil[Fin[Nat.0.suc.suc]])))
    matrix_det_list[R](Nat.0.suc.suc, a,
        List.cons(fin_zero(Nat.0.suc), List.cons(fin_succ(Nat.0.suc, fin_zero(Nat.0)), List.nil[Fin[Nat.0.suc.suc]])),
        List.cons(fin_zero(Nat.0.suc), List.cons(fin_succ(Nat.0.suc, fin_zero(Nat.0)), List.nil[Fin[Nat.0.suc.suc]]))) =
        determinant_list[R, Fin[Nat.0.suc.suc]](square_matrix_entry_function[R](Nat.0.suc.suc, a),
            two_row_list[Fin[Nat.0.suc.suc]](fin_zero(Nat.0.suc), fin_succ(Nat.0.suc, fin_zero(Nat.0))),
            two_col_list[Fin[Nat.0.suc.suc]](fin_zero(Nat.0.suc), fin_succ(Nat.0.suc, fin_zero(Nat.0))))
    matrix_det_suc[R](Nat.0.suc, a) =
        determinant_list[R, Fin[Nat.0.suc.suc]](square_matrix_entry_function[R](Nat.0.suc.suc, a),
            two_row_list[Fin[Nat.0.suc.suc]](fin_zero(Nat.0.suc), fin_succ(Nat.0.suc, fin_zero(Nat.0))),
            two_col_list[Fin[Nat.0.suc.suc]](fin_zero(Nat.0.suc), fin_succ(Nat.0.suc, fin_zero(Nat.0))))
}

/// The canonical determinant of a two-by-two matrix is the difference of the
/// diagonal and off-diagonal products, at the `Nat.0.suc.suc` representation.
theorem matrix_det_suc_two_by_two_zero[R: Ring](a: Matrix[R, Nat.0.suc.suc, Nat.0.suc.suc]) {
    matrix_det_suc[R](Nat.0.suc, a) =
        matrix_entry[R](Nat.0.suc.suc, Nat.0.suc.suc, a, fin_zero(Nat.0.suc), fin_zero(Nat.0.suc)) *
            matrix_entry[R](Nat.0.suc.suc, Nat.0.suc.suc, a, fin_succ(Nat.0.suc, fin_zero(Nat.0)), fin_succ(Nat.0.suc, fin_zero(Nat.0))) -
        matrix_entry[R](Nat.0.suc.suc, Nat.0.suc.suc, a, fin_zero(Nat.0.suc), fin_succ(Nat.0.suc, fin_zero(Nat.0))) *
            matrix_entry[R](Nat.0.suc.suc, Nat.0.suc.suc, a, fin_succ(Nat.0.suc, fin_zero(Nat.0)), fin_zero(Nat.0.suc))
} by {
    matrix_det_suc_two_bridge[R](a)
    matrix_det_suc[R](Nat.0.suc, a) =
        determinant_list[R, Fin[Nat.0.suc.suc]](square_matrix_entry_function[R](Nat.0.suc.suc, a),
            two_row_list[Fin[Nat.0.suc.suc]](fin_zero(Nat.0.suc), fin_succ(Nat.0.suc, fin_zero(Nat.0))),
            two_col_list[Fin[Nat.0.suc.suc]](fin_zero(Nat.0.suc), fin_succ(Nat.0.suc, fin_zero(Nat.0))))
    fin_zero_value(Nat.0.suc)
    fin_zero(Nat.0.suc).value = Nat.0
    fin_succ_value(Nat.0.suc, fin_zero(Nat.0))
    fin_succ(Nat.0.suc, fin_zero(Nat.0)).value = Nat.0.suc
    Nat.0 != Nat.0.suc
    if fin_zero(Nat.0.suc) = fin_succ(Nat.0.suc, fin_zero(Nat.0)) {
        fin_zero(Nat.0.suc).value = fin_succ(Nat.0.suc, fin_zero(Nat.0)).value
        Nat.0 = Nat.0.suc
        false
    }
    fin_zero(Nat.0.suc) != fin_succ(Nat.0.suc, fin_zero(Nat.0))
    determinant_list_two_two[R, Fin[Nat.0.suc.suc]](square_matrix_entry_function[R](Nat.0.suc.suc, a),
        fin_zero(Nat.0.suc), fin_succ(Nat.0.suc, fin_zero(Nat.0)),
        fin_zero(Nat.0.suc), fin_succ(Nat.0.suc, fin_zero(Nat.0)))
    determinant_list[R, Fin[Nat.0.suc.suc]](square_matrix_entry_function[R](Nat.0.suc.suc, a),
        two_row_list[Fin[Nat.0.suc.suc]](fin_zero(Nat.0.suc), fin_succ(Nat.0.suc, fin_zero(Nat.0))),
        two_col_list[Fin[Nat.0.suc.suc]](fin_zero(Nat.0.suc), fin_succ(Nat.0.suc, fin_zero(Nat.0)))) =
        square_matrix_entry_function[R](Nat.0.suc.suc, a, fin_zero(Nat.0.suc), fin_zero(Nat.0.suc)) *
            square_matrix_entry_function[R](Nat.0.suc.suc, a, fin_succ(Nat.0.suc, fin_zero(Nat.0)), fin_succ(Nat.0.suc, fin_zero(Nat.0))) -
        square_matrix_entry_function[R](Nat.0.suc.suc, a, fin_zero(Nat.0.suc), fin_succ(Nat.0.suc, fin_zero(Nat.0))) *
            square_matrix_entry_function[R](Nat.0.suc.suc, a, fin_succ(Nat.0.suc, fin_zero(Nat.0)), fin_zero(Nat.0.suc))
    matrix_det_suc[R](Nat.0.suc, a) =
        square_matrix_entry_function[R](Nat.0.suc.suc, a, fin_zero(Nat.0.suc), fin_zero(Nat.0.suc)) *
            square_matrix_entry_function[R](Nat.0.suc.suc, a, fin_succ(Nat.0.suc, fin_zero(Nat.0)), fin_succ(Nat.0.suc, fin_zero(Nat.0))) -
        square_matrix_entry_function[R](Nat.0.suc.suc, a, fin_zero(Nat.0.suc), fin_succ(Nat.0.suc, fin_zero(Nat.0))) *
            square_matrix_entry_function[R](Nat.0.suc.suc, a, fin_succ(Nat.0.suc, fin_zero(Nat.0)), fin_zero(Nat.0.suc))
    square_matrix_entry_function[R](Nat.0.suc.suc, a, fin_zero(Nat.0.suc), fin_zero(Nat.0.suc)) =
        matrix_entry[R](Nat.0.suc.suc, Nat.0.suc.suc, a, fin_zero(Nat.0.suc), fin_zero(Nat.0.suc))
    square_matrix_entry_function[R](Nat.0.suc.suc, a, fin_succ(Nat.0.suc, fin_zero(Nat.0)), fin_succ(Nat.0.suc, fin_zero(Nat.0))) =
        matrix_entry[R](Nat.0.suc.suc, Nat.0.suc.suc, a, fin_succ(Nat.0.suc, fin_zero(Nat.0)), fin_succ(Nat.0.suc, fin_zero(Nat.0)))
    square_matrix_entry_function[R](Nat.0.suc.suc, a, fin_zero(Nat.0.suc), fin_succ(Nat.0.suc, fin_zero(Nat.0))) =
        matrix_entry[R](Nat.0.suc.suc, Nat.0.suc.suc, a, fin_zero(Nat.0.suc), fin_succ(Nat.0.suc, fin_zero(Nat.0)))
    square_matrix_entry_function[R](Nat.0.suc.suc, a, fin_succ(Nat.0.suc, fin_zero(Nat.0)), fin_zero(Nat.0.suc)) =
        matrix_entry[R](Nat.0.suc.suc, Nat.0.suc.suc, a, fin_succ(Nat.0.suc, fin_zero(Nat.0)), fin_zero(Nat.0.suc))
    matrix_det_suc[R](Nat.0.suc, a) =
        matrix_entry[R](Nat.0.suc.suc, Nat.0.suc.suc, a, fin_zero(Nat.0.suc), fin_zero(Nat.0.suc)) *
            matrix_entry[R](Nat.0.suc.suc, Nat.0.suc.suc, a, fin_succ(Nat.0.suc, fin_zero(Nat.0)), fin_succ(Nat.0.suc, fin_zero(Nat.0))) -
        matrix_entry[R](Nat.0.suc.suc, Nat.0.suc.suc, a, fin_zero(Nat.0.suc), fin_succ(Nat.0.suc, fin_zero(Nat.0))) *
            matrix_entry[R](Nat.0.suc.suc, Nat.0.suc.suc, a, fin_succ(Nat.0.suc, fin_zero(Nat.0)), fin_zero(Nat.0.suc))
}

/// An entry of `A - lam * I` at the upper-left corner is `a00 - lam`.
theorem char_poly_matrix_entry_00[R: Ring](a: Matrix[R, Nat.0.suc.suc, Nat.0.suc.suc], lam: R) {
    matrix_entry[R](Nat.0.suc.suc, Nat.0.suc.suc,
        matrix_sub[R](Nat.0.suc.suc, Nat.0.suc.suc, a,
            matrix_smul[R, R](ring_as_module[R], Nat.0.suc.suc, Nat.0.suc.suc, lam, matrix_one[R](Nat.0.suc.suc))),
        fin_zero(Nat.0.suc), fin_zero(Nat.0.suc)) =
        matrix2_00[R](Nat.0, a) - lam
} by {
    matrix_entry_sub[R](Nat.0.suc.suc, Nat.0.suc.suc, a,
        matrix_smul[R, R](ring_as_module[R], Nat.0.suc.suc, Nat.0.suc.suc, lam, matrix_one[R](Nat.0.suc.suc)),
        fin_zero(Nat.0.suc), fin_zero(Nat.0.suc))
    matrix_entry[R](Nat.0.suc.suc, Nat.0.suc.suc, matrix_sub[R](Nat.0.suc.suc, Nat.0.suc.suc, a,
        matrix_smul[R, R](ring_as_module[R], Nat.0.suc.suc, Nat.0.suc.suc, lam, matrix_one[R](Nat.0.suc.suc))),
        fin_zero(Nat.0.suc), fin_zero(Nat.0.suc)) =
        matrix_entry[R](Nat.0.suc.suc, Nat.0.suc.suc, a, fin_zero(Nat.0.suc), fin_zero(Nat.0.suc)) -
        matrix_entry[R](Nat.0.suc.suc, Nat.0.suc.suc,
            matrix_smul[R, R](ring_as_module[R], Nat.0.suc.suc, Nat.0.suc.suc, lam, matrix_one[R](Nat.0.suc.suc)),
            fin_zero(Nat.0.suc), fin_zero(Nat.0.suc))
    matrix2_00[R](Nat.0, a) = matrix_entry[R](Nat.0.suc.suc, Nat.0.suc.suc, a, fin_zero(Nat.0.suc), fin_zero(Nat.0.suc))
    matrix_entry_smul[R, R](ring_as_module[R], Nat.0.suc.suc, Nat.0.suc.suc, lam, matrix_one[R](Nat.0.suc.suc),
        fin_zero(Nat.0.suc), fin_zero(Nat.0.suc))
    matrix_entry[R](Nat.0.suc.suc, Nat.0.suc.suc,
        matrix_smul[R, R](ring_as_module[R], Nat.0.suc.suc, Nat.0.suc.suc, lam, matrix_one[R](Nat.0.suc.suc)),
        fin_zero(Nat.0.suc), fin_zero(Nat.0.suc)) =
        ring_as_module[R].smul(lam, matrix_entry[R](Nat.0.suc.suc, Nat.0.suc.suc, matrix_one[R](Nat.0.suc.suc),
            fin_zero(Nat.0.suc), fin_zero(Nat.0.suc)))
    ring_as_module_smul[R](lam, matrix_entry[R](Nat.0.suc.suc, Nat.0.suc.suc, matrix_one[R](Nat.0.suc.suc),
        fin_zero(Nat.0.suc), fin_zero(Nat.0.suc)))
    matrix_entry_one_diag[R](Nat.0.suc.suc, fin_zero(Nat.0.suc))
    matrix_entry[R](Nat.0.suc.suc, Nat.0.suc.suc, matrix_one[R](Nat.0.suc.suc),
        fin_zero(Nat.0.suc), fin_zero(Nat.0.suc)) = R.1
    matrix_entry[R](Nat.0.suc.suc, Nat.0.suc.suc,
        matrix_sub[R](Nat.0.suc.suc, Nat.0.suc.suc, a,
            matrix_smul[R, R](ring_as_module[R], Nat.0.suc.suc, Nat.0.suc.suc, lam, matrix_one[R](Nat.0.suc.suc))),
        fin_zero(Nat.0.suc), fin_zero(Nat.0.suc)) =
        matrix2_00[R](Nat.0, a) - lam * R.1
    matrix_entry[R](Nat.0.suc.suc, Nat.0.suc.suc,
        matrix_sub[R](Nat.0.suc.suc, Nat.0.suc.suc, a,
            matrix_smul[R, R](ring_as_module[R], Nat.0.suc.suc, Nat.0.suc.suc, lam, matrix_one[R](Nat.0.suc.suc))),
        fin_zero(Nat.0.suc), fin_zero(Nat.0.suc)) = matrix2_00[R](Nat.0, a) - lam
}

/// An entry of `A - lam * I` at the upper-right corner is `a01`.
theorem char_poly_matrix_entry_01[R: Ring](a: Matrix[R, Nat.0.suc.suc, Nat.0.suc.suc], lam: R) {
    matrix_entry[R](Nat.0.suc.suc, Nat.0.suc.suc,
        matrix_sub[R](Nat.0.suc.suc, Nat.0.suc.suc, a,
            matrix_smul[R, R](ring_as_module[R], Nat.0.suc.suc, Nat.0.suc.suc, lam, matrix_one[R](Nat.0.suc.suc))),
        fin_zero(Nat.0.suc), fin_succ(Nat.0.suc, fin_zero(Nat.0))) = matrix2_01[R](Nat.0, a)
} by {
    matrix_entry_sub[R](Nat.0.suc.suc, Nat.0.suc.suc, a,
        matrix_smul[R, R](ring_as_module[R], Nat.0.suc.suc, Nat.0.suc.suc, lam, matrix_one[R](Nat.0.suc.suc)),
        fin_zero(Nat.0.suc), fin_succ(Nat.0.suc, fin_zero(Nat.0)))
    matrix_entry[R](Nat.0.suc.suc, Nat.0.suc.suc, matrix_sub[R](Nat.0.suc.suc, Nat.0.suc.suc, a,
        matrix_smul[R, R](ring_as_module[R], Nat.0.suc.suc, Nat.0.suc.suc, lam, matrix_one[R](Nat.0.suc.suc))),
        fin_zero(Nat.0.suc), fin_succ(Nat.0.suc, fin_zero(Nat.0))) =
        matrix_entry[R](Nat.0.suc.suc, Nat.0.suc.suc, a, fin_zero(Nat.0.suc), fin_succ(Nat.0.suc, fin_zero(Nat.0))) -
        matrix_entry[R](Nat.0.suc.suc, Nat.0.suc.suc,
            matrix_smul[R, R](ring_as_module[R], Nat.0.suc.suc, Nat.0.suc.suc, lam, matrix_one[R](Nat.0.suc.suc)),
            fin_zero(Nat.0.suc), fin_succ(Nat.0.suc, fin_zero(Nat.0)))
    matrix2_01[R](Nat.0, a) = matrix_entry[R](Nat.0.suc.suc, Nat.0.suc.suc, a, fin_zero(Nat.0.suc), fin_succ(Nat.0.suc, fin_zero(Nat.0)))
    matrix_entry_smul[R, R](ring_as_module[R], Nat.0.suc.suc, Nat.0.suc.suc, lam, matrix_one[R](Nat.0.suc.suc),
        fin_zero(Nat.0.suc), fin_succ(Nat.0.suc, fin_zero(Nat.0)))
    matrix_entry[R](Nat.0.suc.suc, Nat.0.suc.suc,
        matrix_smul[R, R](ring_as_module[R], Nat.0.suc.suc, Nat.0.suc.suc, lam, matrix_one[R](Nat.0.suc.suc)),
        fin_zero(Nat.0.suc), fin_succ(Nat.0.suc, fin_zero(Nat.0))) =
        ring_as_module[R].smul(lam, matrix_entry[R](Nat.0.suc.suc, Nat.0.suc.suc, matrix_one[R](Nat.0.suc.suc),
            fin_zero(Nat.0.suc), fin_succ(Nat.0.suc, fin_zero(Nat.0))))
    ring_as_module_smul[R](lam, matrix_entry[R](Nat.0.suc.suc, Nat.0.suc.suc, matrix_one[R](Nat.0.suc.suc),
        fin_zero(Nat.0.suc), fin_succ(Nat.0.suc, fin_zero(Nat.0))))
    matrix_entry_one_off_diag[R](Nat.0.suc.suc, fin_zero(Nat.0.suc), fin_succ(Nat.0.suc, fin_zero(Nat.0)))
    matrix_entry[R](Nat.0.suc.suc, Nat.0.suc.suc, matrix_one[R](Nat.0.suc.suc),
        fin_zero(Nat.0.suc), fin_succ(Nat.0.suc, fin_zero(Nat.0))) = R.0
    matrix_entry[R](Nat.0.suc.suc, Nat.0.suc.suc,
        matrix_sub[R](Nat.0.suc.suc, Nat.0.suc.suc, a,
            matrix_smul[R, R](ring_as_module[R], Nat.0.suc.suc, Nat.0.suc.suc, lam, matrix_one[R](Nat.0.suc.suc))),
        fin_zero(Nat.0.suc), fin_succ(Nat.0.suc, fin_zero(Nat.0))) = matrix2_01[R](Nat.0, a) - lam * R.0
    mul_zero_right[R](lam)
    lam * R.0 = R.0
    matrix_entry[R](Nat.0.suc.suc, Nat.0.suc.suc,
        matrix_sub[R](Nat.0.suc.suc, Nat.0.suc.suc, a,
            matrix_smul[R, R](ring_as_module[R], Nat.0.suc.suc, Nat.0.suc.suc, lam, matrix_one[R](Nat.0.suc.suc))),
        fin_zero(Nat.0.suc), fin_succ(Nat.0.suc, fin_zero(Nat.0))) = matrix2_01[R](Nat.0, a) - R.0
    sub_zero_right[R](matrix2_01[R](Nat.0, a))
    matrix2_01[R](Nat.0, a) - R.0 = matrix2_01[R](Nat.0, a)
    matrix_entry[R](Nat.0.suc.suc, Nat.0.suc.suc,
        matrix_sub[R](Nat.0.suc.suc, Nat.0.suc.suc, a,
            matrix_smul[R, R](ring_as_module[R], Nat.0.suc.suc, Nat.0.suc.suc, lam, matrix_one[R](Nat.0.suc.suc))),
        fin_zero(Nat.0.suc), fin_succ(Nat.0.suc, fin_zero(Nat.0))) = matrix2_01[R](Nat.0, a)
}

/// An entry of `A - lam * I` at the lower-left corner is `a10`.
theorem char_poly_matrix_entry_10[R: Ring](a: Matrix[R, Nat.0.suc.suc, Nat.0.suc.suc], lam: R) {
    matrix_entry[R](Nat.0.suc.suc, Nat.0.suc.suc,
        matrix_sub[R](Nat.0.suc.suc, Nat.0.suc.suc, a,
            matrix_smul[R, R](ring_as_module[R], Nat.0.suc.suc, Nat.0.suc.suc, lam, matrix_one[R](Nat.0.suc.suc))),
        fin_succ(Nat.0.suc, fin_zero(Nat.0)), fin_zero(Nat.0.suc)) = matrix2_10[R](Nat.0, a)
} by {
    matrix_entry_sub[R](Nat.0.suc.suc, Nat.0.suc.suc, a,
        matrix_smul[R, R](ring_as_module[R], Nat.0.suc.suc, Nat.0.suc.suc, lam, matrix_one[R](Nat.0.suc.suc)),
        fin_succ(Nat.0.suc, fin_zero(Nat.0)), fin_zero(Nat.0.suc))
    matrix_entry[R](Nat.0.suc.suc, Nat.0.suc.suc, matrix_sub[R](Nat.0.suc.suc, Nat.0.suc.suc, a,
        matrix_smul[R, R](ring_as_module[R], Nat.0.suc.suc, Nat.0.suc.suc, lam, matrix_one[R](Nat.0.suc.suc))),
        fin_succ(Nat.0.suc, fin_zero(Nat.0)), fin_zero(Nat.0.suc)) =
        matrix_entry[R](Nat.0.suc.suc, Nat.0.suc.suc, a, fin_succ(Nat.0.suc, fin_zero(Nat.0)), fin_zero(Nat.0.suc)) -
        matrix_entry[R](Nat.0.suc.suc, Nat.0.suc.suc,
            matrix_smul[R, R](ring_as_module[R], Nat.0.suc.suc, Nat.0.suc.suc, lam, matrix_one[R](Nat.0.suc.suc)),
            fin_succ(Nat.0.suc, fin_zero(Nat.0)), fin_zero(Nat.0.suc))
    matrix2_10[R](Nat.0, a) = matrix_entry[R](Nat.0.suc.suc, Nat.0.suc.suc, a, fin_succ(Nat.0.suc, fin_zero(Nat.0)), fin_zero(Nat.0.suc))
    matrix_entry_smul[R, R](ring_as_module[R], Nat.0.suc.suc, Nat.0.suc.suc, lam, matrix_one[R](Nat.0.suc.suc),
        fin_succ(Nat.0.suc, fin_zero(Nat.0)), fin_zero(Nat.0.suc))
    matrix_entry[R](Nat.0.suc.suc, Nat.0.suc.suc,
        matrix_smul[R, R](ring_as_module[R], Nat.0.suc.suc, Nat.0.suc.suc, lam, matrix_one[R](Nat.0.suc.suc)),
        fin_succ(Nat.0.suc, fin_zero(Nat.0)), fin_zero(Nat.0.suc)) =
        ring_as_module[R].smul(lam, matrix_entry[R](Nat.0.suc.suc, Nat.0.suc.suc, matrix_one[R](Nat.0.suc.suc),
            fin_succ(Nat.0.suc, fin_zero(Nat.0)), fin_zero(Nat.0.suc)))
    ring_as_module_smul[R](lam, matrix_entry[R](Nat.0.suc.suc, Nat.0.suc.suc, matrix_one[R](Nat.0.suc.suc),
        fin_succ(Nat.0.suc, fin_zero(Nat.0)), fin_zero(Nat.0.suc)))
    matrix_entry_one_off_diag[R](Nat.0.suc.suc, fin_succ(Nat.0.suc, fin_zero(Nat.0)), fin_zero(Nat.0.suc))
    matrix_entry[R](Nat.0.suc.suc, Nat.0.suc.suc, matrix_one[R](Nat.0.suc.suc),
        fin_succ(Nat.0.suc, fin_zero(Nat.0)), fin_zero(Nat.0.suc)) = R.0
    matrix_entry[R](Nat.0.suc.suc, Nat.0.suc.suc,
        matrix_sub[R](Nat.0.suc.suc, Nat.0.suc.suc, a,
            matrix_smul[R, R](ring_as_module[R], Nat.0.suc.suc, Nat.0.suc.suc, lam, matrix_one[R](Nat.0.suc.suc))),
        fin_succ(Nat.0.suc, fin_zero(Nat.0)), fin_zero(Nat.0.suc)) = matrix2_10[R](Nat.0, a) - lam * R.0
    mul_zero_right[R](lam)
    lam * R.0 = R.0
    matrix_entry[R](Nat.0.suc.suc, Nat.0.suc.suc,
        matrix_sub[R](Nat.0.suc.suc, Nat.0.suc.suc, a,
            matrix_smul[R, R](ring_as_module[R], Nat.0.suc.suc, Nat.0.suc.suc, lam, matrix_one[R](Nat.0.suc.suc))),
        fin_succ(Nat.0.suc, fin_zero(Nat.0)), fin_zero(Nat.0.suc)) = matrix2_10[R](Nat.0, a) - R.0
    sub_zero_right[R](matrix2_10[R](Nat.0, a))
    matrix2_10[R](Nat.0, a) - R.0 = matrix2_10[R](Nat.0, a)
    matrix_entry[R](Nat.0.suc.suc, Nat.0.suc.suc,
        matrix_sub[R](Nat.0.suc.suc, Nat.0.suc.suc, a,
            matrix_smul[R, R](ring_as_module[R], Nat.0.suc.suc, Nat.0.suc.suc, lam, matrix_one[R](Nat.0.suc.suc))),
        fin_succ(Nat.0.suc, fin_zero(Nat.0)), fin_zero(Nat.0.suc)) = matrix2_10[R](Nat.0, a)
}

/// An entry of `A - lam * I` at the lower-right corner is `a11 - lam`.
theorem char_poly_matrix_entry_11[R: Ring](a: Matrix[R, Nat.0.suc.suc, Nat.0.suc.suc], lam: R) {
    matrix_entry[R](Nat.0.suc.suc, Nat.0.suc.suc,
        matrix_sub[R](Nat.0.suc.suc, Nat.0.suc.suc, a,
            matrix_smul[R, R](ring_as_module[R], Nat.0.suc.suc, Nat.0.suc.suc, lam, matrix_one[R](Nat.0.suc.suc))),
        fin_succ(Nat.0.suc, fin_zero(Nat.0)), fin_succ(Nat.0.suc, fin_zero(Nat.0))) =
        matrix2_11[R](Nat.0, a) - lam
} by {
    matrix_entry_sub[R](Nat.0.suc.suc, Nat.0.suc.suc, a,
        matrix_smul[R, R](ring_as_module[R], Nat.0.suc.suc, Nat.0.suc.suc, lam, matrix_one[R](Nat.0.suc.suc)),
        fin_succ(Nat.0.suc, fin_zero(Nat.0)), fin_succ(Nat.0.suc, fin_zero(Nat.0)))
    matrix_entry[R](Nat.0.suc.suc, Nat.0.suc.suc, matrix_sub[R](Nat.0.suc.suc, Nat.0.suc.suc, a,
        matrix_smul[R, R](ring_as_module[R], Nat.0.suc.suc, Nat.0.suc.suc, lam, matrix_one[R](Nat.0.suc.suc))),
        fin_succ(Nat.0.suc, fin_zero(Nat.0)), fin_succ(Nat.0.suc, fin_zero(Nat.0))) =
        matrix_entry[R](Nat.0.suc.suc, Nat.0.suc.suc, a, fin_succ(Nat.0.suc, fin_zero(Nat.0)), fin_succ(Nat.0.suc, fin_zero(Nat.0))) -
        matrix_entry[R](Nat.0.suc.suc, Nat.0.suc.suc,
            matrix_smul[R, R](ring_as_module[R], Nat.0.suc.suc, Nat.0.suc.suc, lam, matrix_one[R](Nat.0.suc.suc)),
            fin_succ(Nat.0.suc, fin_zero(Nat.0)), fin_succ(Nat.0.suc, fin_zero(Nat.0)))
    matrix2_11[R](Nat.0, a) = matrix_entry[R](Nat.0.suc.suc, Nat.0.suc.suc, a, fin_succ(Nat.0.suc, fin_zero(Nat.0)), fin_succ(Nat.0.suc, fin_zero(Nat.0)))
    matrix_entry_smul[R, R](ring_as_module[R], Nat.0.suc.suc, Nat.0.suc.suc, lam, matrix_one[R](Nat.0.suc.suc),
        fin_succ(Nat.0.suc, fin_zero(Nat.0)), fin_succ(Nat.0.suc, fin_zero(Nat.0)))
    matrix_entry[R](Nat.0.suc.suc, Nat.0.suc.suc,
        matrix_smul[R, R](ring_as_module[R], Nat.0.suc.suc, Nat.0.suc.suc, lam, matrix_one[R](Nat.0.suc.suc)),
        fin_succ(Nat.0.suc, fin_zero(Nat.0)), fin_succ(Nat.0.suc, fin_zero(Nat.0))) =
        ring_as_module[R].smul(lam, matrix_entry[R](Nat.0.suc.suc, Nat.0.suc.suc, matrix_one[R](Nat.0.suc.suc),
            fin_succ(Nat.0.suc, fin_zero(Nat.0)), fin_succ(Nat.0.suc, fin_zero(Nat.0))))
    ring_as_module_smul[R](lam, matrix_entry[R](Nat.0.suc.suc, Nat.0.suc.suc, matrix_one[R](Nat.0.suc.suc),
        fin_succ(Nat.0.suc, fin_zero(Nat.0)), fin_succ(Nat.0.suc, fin_zero(Nat.0))))
    matrix_entry_one_diag[R](Nat.0.suc.suc, fin_succ(Nat.0.suc, fin_zero(Nat.0)))
    matrix_entry[R](Nat.0.suc.suc, Nat.0.suc.suc, matrix_one[R](Nat.0.suc.suc),
        fin_succ(Nat.0.suc, fin_zero(Nat.0)), fin_succ(Nat.0.suc, fin_zero(Nat.0))) = R.1
    matrix_entry[R](Nat.0.suc.suc, Nat.0.suc.suc,
        matrix_sub[R](Nat.0.suc.suc, Nat.0.suc.suc, a,
            matrix_smul[R, R](ring_as_module[R], Nat.0.suc.suc, Nat.0.suc.suc, lam, matrix_one[R](Nat.0.suc.suc))),
        fin_succ(Nat.0.suc, fin_zero(Nat.0)), fin_succ(Nat.0.suc, fin_zero(Nat.0))) =
        matrix2_11[R](Nat.0, a) - lam * R.1
    matrix_entry[R](Nat.0.suc.suc, Nat.0.suc.suc,
        matrix_sub[R](Nat.0.suc.suc, Nat.0.suc.suc, a,
            matrix_smul[R, R](ring_as_module[R], Nat.0.suc.suc, Nat.0.suc.suc, lam, matrix_one[R](Nat.0.suc.suc))),
        fin_succ(Nat.0.suc, fin_zero(Nat.0)), fin_succ(Nat.0.suc, fin_zero(Nat.0))) =
        matrix2_11[R](Nat.0, a) - lam
}

/// The characteristic polynomial as a determinant agrees with the entrywise
/// characteristic polynomial of a two-by-two matrix.
theorem char_poly_det_eq_entries[R: Ring](a: Matrix[R, Nat.0.suc.suc, Nat.0.suc.suc], lam: R) {
    char_poly_det[R](Nat.0, a, lam) = char_poly_2x2[R](Nat.0, a, lam)
} by {
    char_poly_det_unfold[R](Nat.0, a, lam)
    char_poly_det[R](Nat.0, a, lam) = matrix_det_suc[R](Nat.0.suc, char_poly_matrix[R](Nat.0, a, lam))
    char_poly_matrix[R](Nat.0, a, lam) = matrix_sub[R](Nat.0.suc.suc, Nat.0.suc.suc, a,
        matrix_smul[R, R](ring_as_module[R], Nat.0.suc.suc, Nat.0.suc.suc, lam, matrix_one[R](Nat.0.suc.suc)))
    matrix_det_suc_two_by_two_zero[R](char_poly_matrix[R](Nat.0, a, lam))
    matrix_det_suc[R](Nat.0.suc, char_poly_matrix[R](Nat.0, a, lam)) =
        matrix_entry[R](Nat.0.suc.suc, Nat.0.suc.suc, char_poly_matrix[R](Nat.0, a, lam), fin_zero(Nat.0.suc), fin_zero(Nat.0.suc)) *
            matrix_entry[R](Nat.0.suc.suc, Nat.0.suc.suc, char_poly_matrix[R](Nat.0, a, lam), fin_succ(Nat.0.suc, fin_zero(Nat.0)), fin_succ(Nat.0.suc, fin_zero(Nat.0))) -
        matrix_entry[R](Nat.0.suc.suc, Nat.0.suc.suc, char_poly_matrix[R](Nat.0, a, lam), fin_zero(Nat.0.suc), fin_succ(Nat.0.suc, fin_zero(Nat.0))) *
            matrix_entry[R](Nat.0.suc.suc, Nat.0.suc.suc, char_poly_matrix[R](Nat.0, a, lam), fin_succ(Nat.0.suc, fin_zero(Nat.0)), fin_zero(Nat.0.suc))
    char_poly_matrix_entry_00[R](a, lam)
    char_poly_matrix_entry_01[R](a, lam)
    char_poly_matrix_entry_10[R](a, lam)
    char_poly_matrix_entry_11[R](a, lam)
    char_poly_det[R](Nat.0, a, lam) =
        (matrix2_00[R](Nat.0, a) - lam) * (matrix2_11[R](Nat.0, a) - lam) -
            matrix2_01[R](Nat.0, a) * matrix2_10[R](Nat.0, a)
    char_poly_2x2[R](Nat.0, a, lam) = (matrix2_00[R](Nat.0, a) - lam) * (matrix2_11[R](Nat.0, a) - lam) -
        matrix2_01[R](Nat.0, a) * matrix2_10[R](Nat.0, a)
    char_poly_det[R](Nat.0, a, lam) = char_poly_2x2[R](Nat.0, a, lam)
}

/// The negation of a sum is the sum of the negations.
theorem ring_neg_sum[R: Ring](x: R, y: R) {
    -x - y = -(x + y)
}

/// Pulling a common trailing summand out of a difference chain.
theorem ring_chain_regroup[R: Ring](x: R, y: R, z: R, w: R) {
    x - y - z + w = x - (y + z) + w
} by {
    x - y - z + w = x + -y + -z + w
    x - (y + z) + w = x + -(y + z) + w
    ring_neg_sum[R](y, z)
    -y - z = -(y + z)
    -y + -z = -(y + z)
    x + -y + -z + w = x + (-y + -z) + w
    x + (-y + -z) + w = x + (-(y + z)) + w
    x + -y + -z + w = x + -(y + z) + w
    x - y - z + w = x - (y + z) + w
}

/// A sum of a left and a right scalar multiple factors as the scalar multiple
/// of the sum.
theorem ring_comm_sums[R: CommRing](a: R, b: R, lam: R) {
    a * lam + lam * b = (a + b) * lam
}

/// The entrywise characteristic polynomial of a two-by-two matrix expands to
/// the standard form `lam^2 - trace(A) * lam + det(A)`.
theorem char_poly_standard_expand[R: CommRing](n: Nat, a: Matrix[R, n.suc.suc, n.suc.suc], lam: R) {
    char_poly_2x2[R](n, a, lam) = char_poly_2x2_standard[R](n, a, lam)
} by {
    ring_sub_mul_sub[R](matrix2_00[R](n, a), lam, matrix2_11[R](n, a), lam)
    (matrix2_00[R](n, a) - lam) * (matrix2_11[R](n, a) - lam) =
        matrix2_00[R](n, a) * matrix2_11[R](n, a) - matrix2_00[R](n, a) * lam -
            lam * matrix2_11[R](n, a) + lam * lam
    char_poly_2x2[R](n, a, lam) =
        matrix2_00[R](n, a) * matrix2_11[R](n, a) - matrix2_00[R](n, a) * lam -
            lam * matrix2_11[R](n, a) + lam * lam - matrix2_01[R](n, a) * matrix2_10[R](n, a)
    ring4_sub_swap[R](matrix2_00[R](n, a) * matrix2_11[R](n, a), matrix2_00[R](n, a) * lam,
        lam * matrix2_11[R](n, a), lam * lam)
    matrix2_00[R](n, a) * matrix2_11[R](n, a) - matrix2_00[R](n, a) * lam -
        lam * matrix2_11[R](n, a) + lam * lam =
        matrix2_00[R](n, a) * matrix2_11[R](n, a) - lam * matrix2_11[R](n, a) -
            matrix2_00[R](n, a) * lam + lam * lam
    ring_sub_add_shift[R](lam * lam, matrix2_00[R](n, a) * matrix2_11[R](n, a),
        lam * matrix2_11[R](n, a), matrix2_00[R](n, a) * lam)
    lam * lam + matrix2_00[R](n, a) * matrix2_11[R](n, a) - lam * matrix2_11[R](n, a) -
        matrix2_00[R](n, a) * lam =
        lam * lam - lam * matrix2_11[R](n, a) - matrix2_00[R](n, a) * lam +
            matrix2_00[R](n, a) * matrix2_11[R](n, a)
    ring4_sub_swap[R](lam * lam, lam * matrix2_11[R](n, a), matrix2_00[R](n, a) * lam,
        matrix2_00[R](n, a) * matrix2_11[R](n, a))
    lam * lam - lam * matrix2_11[R](n, a) - matrix2_00[R](n, a) * lam +
        matrix2_00[R](n, a) * matrix2_11[R](n, a) =
        lam * lam - matrix2_00[R](n, a) * lam - lam * matrix2_11[R](n, a) +
            matrix2_00[R](n, a) * matrix2_11[R](n, a)
    char_poly_2x2[R](n, a, lam) =
        lam * lam - matrix2_00[R](n, a) * lam - lam * matrix2_11[R](n, a) +
            matrix2_00[R](n, a) * matrix2_11[R](n, a) - matrix2_01[R](n, a) * matrix2_10[R](n, a)
    ring_chain_regroup[R](lam * lam, matrix2_00[R](n, a) * lam, lam * matrix2_11[R](n, a),
        matrix2_00[R](n, a) * matrix2_11[R](n, a) - matrix2_01[R](n, a) * matrix2_10[R](n, a))
    lam * lam - matrix2_00[R](n, a) * lam - lam * matrix2_11[R](n, a) +
        (matrix2_00[R](n, a) * matrix2_11[R](n, a) - matrix2_01[R](n, a) * matrix2_10[R](n, a)) =
        lam * lam - (matrix2_00[R](n, a) * lam + lam * matrix2_11[R](n, a)) +
            (matrix2_00[R](n, a) * matrix2_11[R](n, a) - matrix2_01[R](n, a) * matrix2_10[R](n, a))
    matrix2_trace[R](n, a) = matrix2_00[R](n, a) + matrix2_11[R](n, a)
    matrix2_det_entries[R](n, a) =
        matrix2_00[R](n, a) * matrix2_11[R](n, a) - matrix2_01[R](n, a) * matrix2_10[R](n, a)
    char_poly_2x2_standard[R](n, a, lam) =
        lam * lam - (matrix2_00[R](n, a) + matrix2_11[R](n, a)) * lam +
            (matrix2_00[R](n, a) * matrix2_11[R](n, a) - matrix2_01[R](n, a) * matrix2_10[R](n, a))
    ring_comm_sums[R](matrix2_00[R](n, a), matrix2_11[R](n, a), lam)
    matrix2_00[R](n, a) * lam + lam * matrix2_11[R](n, a) =
        (matrix2_00[R](n, a) + matrix2_11[R](n, a)) * lam
    lam * lam - (matrix2_00[R](n, a) * lam + lam * matrix2_11[R](n, a)) +
        (matrix2_00[R](n, a) * matrix2_11[R](n, a) - matrix2_01[R](n, a) * matrix2_10[R](n, a)) =
        lam * lam - (matrix2_00[R](n, a) + matrix2_11[R](n, a)) * lam +
            (matrix2_00[R](n, a) * matrix2_11[R](n, a) - matrix2_01[R](n, a) * matrix2_10[R](n, a))
    char_poly_2x2[R](n, a, lam) =
        lam * lam - (matrix2_00[R](n, a) + matrix2_11[R](n, a)) * lam +
            (matrix2_00[R](n, a) * matrix2_11[R](n, a) - matrix2_01[R](n, a) * matrix2_10[R](n, a))
    char_poly_2x2_standard[R](n, a, lam) =
        lam * lam - (matrix2_00[R](n, a) + matrix2_11[R](n, a)) * lam +
            (matrix2_00[R](n, a) * matrix2_11[R](n, a) - matrix2_01[R](n, a) * matrix2_10[R](n, a))
    char_poly_2x2[R](n, a, lam) = char_poly_2x2_standard[R](n, a, lam)
}

/// The characteristic polynomial as a determinant expands to the standard
/// form `lam^2 - trace(A) * lam + det(A)`.
theorem char_poly_det_eq_standard[R: CommRing](a: Matrix[R, Nat.0.suc.suc, Nat.0.suc.suc], lam: R) {
    char_poly_det[R](Nat.0, a, lam) = char_poly_2x2_standard[R](Nat.0, a, lam)
} by {
    char_poly_det_eq_entries[R](a, lam)
    char_poly_det[R](Nat.0, a, lam) = char_poly_2x2[R](Nat.0, a, lam)
    char_poly_standard_expand[R](Nat.0, a, lam)
    char_poly_2x2[R](Nat.0, a, lam) = char_poly_2x2_standard[R](Nat.0, a, lam)
    char_poly_det[R](Nat.0, a, lam) = char_poly_2x2_standard[R](Nat.0, a, lam)
}

// ---------------------------------------------------------------------------
// The Cayley-Hamilton theorem for two-by-two matrices
// ---------------------------------------------------------------------------

/// `(x - y) + (z - x)` telescopes to `z - y`.
theorem ring_add_sub_cancel[R: Ring](x: R, y: R, z: R) {
    (x - y) + (z - x) = z - y
} by {
    (x - y) + (z - x) = (x + -y) + (z + -x)
    (x + -y) + (z + -x) = (x + -y) + (-x + z)
    add_swap_inner[R](x, -y, -x, z)
    (x + -y) + (-x + z) = (x + -x) + (-y + z)
    (x + -x) + (-y + z) = (x + -x) + (z + -y)
    x + -x = R.0
    (x + -x) + (z + -y) = R.0 + (z + -y)
    R.0 + (z + -y) = z - y
    (x - y) + (z - x) = z - y
}

/// The difference of a product and its swapped factors is zero.
theorem ring_mul_comm_sub_zero[R: CommRing](a: R, b: R) {
    a * b - b * a = R.0
} by {
    b * a = a * b
    a * b - b * a = a * b - a * b
    a * b - a * b = R.0
    a * b - b * a = R.0
}

/// The upper-left entry of the two-by-two Cayley-Hamilton expression is zero.
theorem cayley_hamilton_entry_00[R: CommRing](a: Matrix[R, Nat.0.suc.suc, Nat.0.suc.suc]) {
    matrix_entry[R](Nat.0.suc.suc, Nat.0.suc.suc, cayley_hamilton_lhs[R](Nat.0, a),
        fin_zero(Nat.0.suc), fin_zero(Nat.0.suc)) = R.0
} by {
    cayley_hamilton_lhs[R](Nat.0, a) = square_matrix_mul[R](Nat.0.suc.suc, a, a) -
        matrix_smul[R, R](ring_as_module[R], Nat.0.suc.suc, Nat.0.suc.suc, matrix2_trace[R](Nat.0, a), a) +
        matrix_smul[R, R](ring_as_module[R], Nat.0.suc.suc, Nat.0.suc.suc, matrix2_det_entries[R](Nat.0, a), matrix_one[R](Nat.0.suc.suc))
    matrix_entry[R](Nat.0.suc.suc, Nat.0.suc.suc, cayley_hamilton_lhs[R](Nat.0, a), fin_zero(Nat.0.suc), fin_zero(Nat.0.suc)) =
        matrix_entry[R](Nat.0.suc.suc, Nat.0.suc.suc,
            matrix_add[R](Nat.0.suc.suc, Nat.0.suc.suc,
                matrix_sub[R](Nat.0.suc.suc, Nat.0.suc.suc, square_matrix_mul[R](Nat.0.suc.suc, a, a),
                    matrix_smul[R, R](ring_as_module[R], Nat.0.suc.suc, Nat.0.suc.suc, matrix2_trace[R](Nat.0, a), a)),
                matrix_smul[R, R](ring_as_module[R], Nat.0.suc.suc, Nat.0.suc.suc, matrix2_det_entries[R](Nat.0, a), matrix_one[R](Nat.0.suc.suc))),
            fin_zero(Nat.0.suc), fin_zero(Nat.0.suc))
    matrix_entry_add[R](Nat.0.suc.suc, Nat.0.suc.suc,
        matrix_sub[R](Nat.0.suc.suc, Nat.0.suc.suc, square_matrix_mul[R](Nat.0.suc.suc, a, a),
            matrix_smul[R, R](ring_as_module[R], Nat.0.suc.suc, Nat.0.suc.suc, matrix2_trace[R](Nat.0, a), a)),
        matrix_smul[R, R](ring_as_module[R], Nat.0.suc.suc, Nat.0.suc.suc, matrix2_det_entries[R](Nat.0, a), matrix_one[R](Nat.0.suc.suc)),
        fin_zero(Nat.0.suc), fin_zero(Nat.0.suc))
    matrix_entry[R](Nat.0.suc.suc, Nat.0.suc.suc,
        matrix_add[R](Nat.0.suc.suc, Nat.0.suc.suc,
            matrix_sub[R](Nat.0.suc.suc, Nat.0.suc.suc, square_matrix_mul[R](Nat.0.suc.suc, a, a),
                matrix_smul[R, R](ring_as_module[R], Nat.0.suc.suc, Nat.0.suc.suc, matrix2_trace[R](Nat.0, a), a)),
            matrix_smul[R, R](ring_as_module[R], Nat.0.suc.suc, Nat.0.suc.suc, matrix2_det_entries[R](Nat.0, a), matrix_one[R](Nat.0.suc.suc))),
        fin_zero(Nat.0.suc), fin_zero(Nat.0.suc)) =
        matrix_entry[R](Nat.0.suc.suc, Nat.0.suc.suc,
            matrix_sub[R](Nat.0.suc.suc, Nat.0.suc.suc, square_matrix_mul[R](Nat.0.suc.suc, a, a),
                matrix_smul[R, R](ring_as_module[R], Nat.0.suc.suc, Nat.0.suc.suc, matrix2_trace[R](Nat.0, a), a)),
            fin_zero(Nat.0.suc), fin_zero(Nat.0.suc)) +
        matrix_entry[R](Nat.0.suc.suc, Nat.0.suc.suc,
            matrix_smul[R, R](ring_as_module[R], Nat.0.suc.suc, Nat.0.suc.suc, matrix2_det_entries[R](Nat.0, a), matrix_one[R](Nat.0.suc.suc)),
            fin_zero(Nat.0.suc), fin_zero(Nat.0.suc))
    matrix_entry_sub[R](Nat.0.suc.suc, Nat.0.suc.suc, square_matrix_mul[R](Nat.0.suc.suc, a, a),
        matrix_smul[R, R](ring_as_module[R], Nat.0.suc.suc, Nat.0.suc.suc, matrix2_trace[R](Nat.0, a), a),
        fin_zero(Nat.0.suc), fin_zero(Nat.0.suc))
    matrix_entry[R](Nat.0.suc.suc, Nat.0.suc.suc,
        matrix_sub[R](Nat.0.suc.suc, Nat.0.suc.suc, square_matrix_mul[R](Nat.0.suc.suc, a, a),
            matrix_smul[R, R](ring_as_module[R], Nat.0.suc.suc, Nat.0.suc.suc, matrix2_trace[R](Nat.0, a), a)),
        fin_zero(Nat.0.suc), fin_zero(Nat.0.suc)) =
        matrix_entry[R](Nat.0.suc.suc, Nat.0.suc.suc, square_matrix_mul[R](Nat.0.suc.suc, a, a), fin_zero(Nat.0.suc), fin_zero(Nat.0.suc)) -
        matrix_entry[R](Nat.0.suc.suc, Nat.0.suc.suc,
            matrix_smul[R, R](ring_as_module[R], Nat.0.suc.suc, Nat.0.suc.suc, matrix2_trace[R](Nat.0, a), a),
            fin_zero(Nat.0.suc), fin_zero(Nat.0.suc))
    matrix2_mul_entry_00[R](a, a)
    matrix_entry[R](Nat.0.suc.suc, Nat.0.suc.suc, square_matrix_mul[R](Nat.0.suc.suc, a, a), fin_zero(Nat.0.suc), fin_zero(Nat.0.suc)) =
        matrix2_00[R](Nat.0, a) * matrix2_00[R](Nat.0, a) + matrix2_01[R](Nat.0, a) * matrix2_10[R](Nat.0, a)
    matrix_entry_smul[R, R](ring_as_module[R], Nat.0.suc.suc, Nat.0.suc.suc, matrix2_trace[R](Nat.0, a), a,
        fin_zero(Nat.0.suc), fin_zero(Nat.0.suc))
    matrix_entry[R](Nat.0.suc.suc, Nat.0.suc.suc,
        matrix_smul[R, R](ring_as_module[R], Nat.0.suc.suc, Nat.0.suc.suc, matrix2_trace[R](Nat.0, a), a),
        fin_zero(Nat.0.suc), fin_zero(Nat.0.suc)) =
        ring_as_module[R].smul(matrix2_trace[R](Nat.0, a), matrix_entry[R](Nat.0.suc.suc, Nat.0.suc.suc, a, fin_zero(Nat.0.suc), fin_zero(Nat.0.suc)))
    ring_as_module_smul[R](matrix2_trace[R](Nat.0, a), matrix_entry[R](Nat.0.suc.suc, Nat.0.suc.suc, a, fin_zero(Nat.0.suc), fin_zero(Nat.0.suc)))
    matrix2_trace[R](Nat.0, a) = matrix2_00[R](Nat.0, a) + matrix2_11[R](Nat.0, a)
    matrix_entry[R](Nat.0.suc.suc, Nat.0.suc.suc,
        matrix_smul[R, R](ring_as_module[R], Nat.0.suc.suc, Nat.0.suc.suc, matrix2_trace[R](Nat.0, a), a),
        fin_zero(Nat.0.suc), fin_zero(Nat.0.suc)) = (matrix2_00[R](Nat.0, a) + matrix2_11[R](Nat.0, a)) * matrix2_00[R](Nat.0, a)
    matrix_entry_smul[R, R](ring_as_module[R], Nat.0.suc.suc, Nat.0.suc.suc, matrix2_det_entries[R](Nat.0, a), matrix_one[R](Nat.0.suc.suc),
        fin_zero(Nat.0.suc), fin_zero(Nat.0.suc))
    matrix_entry[R](Nat.0.suc.suc, Nat.0.suc.suc,
        matrix_smul[R, R](ring_as_module[R], Nat.0.suc.suc, Nat.0.suc.suc, matrix2_det_entries[R](Nat.0, a), matrix_one[R](Nat.0.suc.suc)),
        fin_zero(Nat.0.suc), fin_zero(Nat.0.suc)) =
        ring_as_module[R].smul(matrix2_det_entries[R](Nat.0, a), matrix_entry[R](Nat.0.suc.suc, Nat.0.suc.suc, matrix_one[R](Nat.0.suc.suc), fin_zero(Nat.0.suc), fin_zero(Nat.0.suc)))
    ring_as_module_smul[R](matrix2_det_entries[R](Nat.0, a), matrix_entry[R](Nat.0.suc.suc, Nat.0.suc.suc, matrix_one[R](Nat.0.suc.suc), fin_zero(Nat.0.suc), fin_zero(Nat.0.suc)))
    matrix_entry_one_diag[R](Nat.0.suc.suc, fin_zero(Nat.0.suc))
    matrix_entry[R](Nat.0.suc.suc, Nat.0.suc.suc, matrix_one[R](Nat.0.suc.suc), fin_zero(Nat.0.suc), fin_zero(Nat.0.suc)) = R.1
    matrix2_det_entries[R](Nat.0, a) =
        matrix2_00[R](Nat.0, a) * matrix2_11[R](Nat.0, a) - matrix2_01[R](Nat.0, a) * matrix2_10[R](Nat.0, a)
    matrix_entry[R](Nat.0.suc.suc, Nat.0.suc.suc, cayley_hamilton_lhs[R](Nat.0, a), fin_zero(Nat.0.suc), fin_zero(Nat.0.suc)) =
        matrix_entry[R](Nat.0.suc.suc, Nat.0.suc.suc, square_matrix_mul[R](Nat.0.suc.suc, a, a), fin_zero(Nat.0.suc), fin_zero(Nat.0.suc)) -
            matrix_entry[R](Nat.0.suc.suc, Nat.0.suc.suc,
                matrix_smul[R, R](ring_as_module[R], Nat.0.suc.suc, Nat.0.suc.suc, matrix2_trace[R](Nat.0, a), a),
                fin_zero(Nat.0.suc), fin_zero(Nat.0.suc)) +
            matrix_entry[R](Nat.0.suc.suc, Nat.0.suc.suc,
                matrix_smul[R, R](ring_as_module[R], Nat.0.suc.suc, Nat.0.suc.suc, matrix2_det_entries[R](Nat.0, a), matrix_one[R](Nat.0.suc.suc)),
                fin_zero(Nat.0.suc), fin_zero(Nat.0.suc))
    matrix_entry[R](Nat.0.suc.suc, Nat.0.suc.suc, cayley_hamilton_lhs[R](Nat.0, a), fin_zero(Nat.0.suc), fin_zero(Nat.0.suc)) =
        (matrix2_00[R](Nat.0, a) * matrix2_00[R](Nat.0, a) + matrix2_01[R](Nat.0, a) * matrix2_10[R](Nat.0, a)) -
            matrix_entry[R](Nat.0.suc.suc, Nat.0.suc.suc,
                matrix_smul[R, R](ring_as_module[R], Nat.0.suc.suc, Nat.0.suc.suc, matrix2_trace[R](Nat.0, a), a),
                fin_zero(Nat.0.suc), fin_zero(Nat.0.suc)) +
            matrix_entry[R](Nat.0.suc.suc, Nat.0.suc.suc,
                matrix_smul[R, R](ring_as_module[R], Nat.0.suc.suc, Nat.0.suc.suc, matrix2_det_entries[R](Nat.0, a), matrix_one[R](Nat.0.suc.suc)),
                fin_zero(Nat.0.suc), fin_zero(Nat.0.suc))
    matrix_entry[R](Nat.0.suc.suc, Nat.0.suc.suc, cayley_hamilton_lhs[R](Nat.0, a), fin_zero(Nat.0.suc), fin_zero(Nat.0.suc)) =
        (matrix2_00[R](Nat.0, a) * matrix2_00[R](Nat.0, a) + matrix2_01[R](Nat.0, a) * matrix2_10[R](Nat.0, a)) -
            (matrix2_00[R](Nat.0, a) + matrix2_11[R](Nat.0, a)) * matrix2_00[R](Nat.0, a) +
            matrix_entry[R](Nat.0.suc.suc, Nat.0.suc.suc,
                matrix_smul[R, R](ring_as_module[R], Nat.0.suc.suc, Nat.0.suc.suc, matrix2_det_entries[R](Nat.0, a), matrix_one[R](Nat.0.suc.suc)),
                fin_zero(Nat.0.suc), fin_zero(Nat.0.suc))
    matrix_entry[R](Nat.0.suc.suc, Nat.0.suc.suc, cayley_hamilton_lhs[R](Nat.0, a), fin_zero(Nat.0.suc), fin_zero(Nat.0.suc)) =
        (matrix2_00[R](Nat.0, a) * matrix2_00[R](Nat.0, a) + matrix2_01[R](Nat.0, a) * matrix2_10[R](Nat.0, a)) -
            (matrix2_00[R](Nat.0, a) + matrix2_11[R](Nat.0, a)) * matrix2_00[R](Nat.0, a) +
            (matrix2_00[R](Nat.0, a) * matrix2_11[R](Nat.0, a) - matrix2_01[R](Nat.0, a) * matrix2_10[R](Nat.0, a))
    (matrix2_00[R](Nat.0, a) + matrix2_11[R](Nat.0, a)) * matrix2_00[R](Nat.0, a) =
        matrix2_00[R](Nat.0, a) * matrix2_00[R](Nat.0, a) + matrix2_11[R](Nat.0, a) * matrix2_00[R](Nat.0, a)
    ring_sub_cancel_common[R](matrix2_00[R](Nat.0, a) * matrix2_00[R](Nat.0, a),
        matrix2_01[R](Nat.0, a) * matrix2_10[R](Nat.0, a),
        matrix2_11[R](Nat.0, a) * matrix2_00[R](Nat.0, a))
    (matrix2_00[R](Nat.0, a) * matrix2_00[R](Nat.0, a) + matrix2_01[R](Nat.0, a) * matrix2_10[R](Nat.0, a)) -
        (matrix2_00[R](Nat.0, a) * matrix2_00[R](Nat.0, a) + matrix2_11[R](Nat.0, a) * matrix2_00[R](Nat.0, a)) =
        matrix2_01[R](Nat.0, a) * matrix2_10[R](Nat.0, a) - matrix2_11[R](Nat.0, a) * matrix2_00[R](Nat.0, a)
    matrix_entry[R](Nat.0.suc.suc, Nat.0.suc.suc, cayley_hamilton_lhs[R](Nat.0, a), fin_zero(Nat.0.suc), fin_zero(Nat.0.suc)) =
        (matrix2_01[R](Nat.0, a) * matrix2_10[R](Nat.0, a) - matrix2_11[R](Nat.0, a) * matrix2_00[R](Nat.0, a)) +
            (matrix2_00[R](Nat.0, a) * matrix2_11[R](Nat.0, a) - matrix2_01[R](Nat.0, a) * matrix2_10[R](Nat.0, a))
    ring_add_sub_cancel[R](matrix2_01[R](Nat.0, a) * matrix2_10[R](Nat.0, a),
        matrix2_11[R](Nat.0, a) * matrix2_00[R](Nat.0, a),
        matrix2_00[R](Nat.0, a) * matrix2_11[R](Nat.0, a))
    (matrix2_01[R](Nat.0, a) * matrix2_10[R](Nat.0, a) - matrix2_11[R](Nat.0, a) * matrix2_00[R](Nat.0, a)) +
        (matrix2_00[R](Nat.0, a) * matrix2_11[R](Nat.0, a) - matrix2_01[R](Nat.0, a) * matrix2_10[R](Nat.0, a)) =
        matrix2_00[R](Nat.0, a) * matrix2_11[R](Nat.0, a) - matrix2_11[R](Nat.0, a) * matrix2_00[R](Nat.0, a)
    ring_mul_comm_sub_zero[R](matrix2_00[R](Nat.0, a), matrix2_11[R](Nat.0, a))
    matrix2_00[R](Nat.0, a) * matrix2_11[R](Nat.0, a) - matrix2_11[R](Nat.0, a) * matrix2_00[R](Nat.0, a) = R.0
    matrix_entry[R](Nat.0.suc.suc, Nat.0.suc.suc, cayley_hamilton_lhs[R](Nat.0, a), fin_zero(Nat.0.suc), fin_zero(Nat.0.suc)) = R.0
}

/// The upper-right entry of the two-by-two Cayley-Hamilton expression is zero.
theorem cayley_hamilton_entry_01[R: CommRing](a: Matrix[R, Nat.0.suc.suc, Nat.0.suc.suc]) {
    matrix_entry[R](Nat.0.suc.suc, Nat.0.suc.suc, cayley_hamilton_lhs[R](Nat.0, a),
        fin_zero(Nat.0.suc), fin_succ(Nat.0.suc, fin_zero(Nat.0))) = R.0
} by {
    matrix_entry[R](Nat.0.suc.suc, Nat.0.suc.suc, cayley_hamilton_lhs[R](Nat.0, a), fin_zero(Nat.0.suc), fin_succ(Nat.0.suc, fin_zero(Nat.0))) =
        matrix_entry[R](Nat.0.suc.suc, Nat.0.suc.suc,
            matrix_add[R](Nat.0.suc.suc, Nat.0.suc.suc,
                matrix_sub[R](Nat.0.suc.suc, Nat.0.suc.suc, square_matrix_mul[R](Nat.0.suc.suc, a, a),
                    matrix_smul[R, R](ring_as_module[R], Nat.0.suc.suc, Nat.0.suc.suc, matrix2_trace[R](Nat.0, a), a)),
                matrix_smul[R, R](ring_as_module[R], Nat.0.suc.suc, Nat.0.suc.suc, matrix2_det_entries[R](Nat.0, a), matrix_one[R](Nat.0.suc.suc))),
            fin_zero(Nat.0.suc), fin_succ(Nat.0.suc, fin_zero(Nat.0)))
    matrix_entry_add[R](Nat.0.suc.suc, Nat.0.suc.suc,
        matrix_sub[R](Nat.0.suc.suc, Nat.0.suc.suc, square_matrix_mul[R](Nat.0.suc.suc, a, a),
            matrix_smul[R, R](ring_as_module[R], Nat.0.suc.suc, Nat.0.suc.suc, matrix2_trace[R](Nat.0, a), a)),
        matrix_smul[R, R](ring_as_module[R], Nat.0.suc.suc, Nat.0.suc.suc, matrix2_det_entries[R](Nat.0, a), matrix_one[R](Nat.0.suc.suc)),
        fin_zero(Nat.0.suc), fin_succ(Nat.0.suc, fin_zero(Nat.0)))
    matrix_entry[R](Nat.0.suc.suc, Nat.0.suc.suc,
        matrix_add[R](Nat.0.suc.suc, Nat.0.suc.suc,
            matrix_sub[R](Nat.0.suc.suc, Nat.0.suc.suc, square_matrix_mul[R](Nat.0.suc.suc, a, a),
                matrix_smul[R, R](ring_as_module[R], Nat.0.suc.suc, Nat.0.suc.suc, matrix2_trace[R](Nat.0, a), a)),
            matrix_smul[R, R](ring_as_module[R], Nat.0.suc.suc, Nat.0.suc.suc, matrix2_det_entries[R](Nat.0, a), matrix_one[R](Nat.0.suc.suc))),
        fin_zero(Nat.0.suc), fin_succ(Nat.0.suc, fin_zero(Nat.0))) =
        matrix_entry[R](Nat.0.suc.suc, Nat.0.suc.suc,
            matrix_sub[R](Nat.0.suc.suc, Nat.0.suc.suc, square_matrix_mul[R](Nat.0.suc.suc, a, a),
                matrix_smul[R, R](ring_as_module[R], Nat.0.suc.suc, Nat.0.suc.suc, matrix2_trace[R](Nat.0, a), a)),
            fin_zero(Nat.0.suc), fin_succ(Nat.0.suc, fin_zero(Nat.0))) +
        matrix_entry[R](Nat.0.suc.suc, Nat.0.suc.suc,
            matrix_smul[R, R](ring_as_module[R], Nat.0.suc.suc, Nat.0.suc.suc, matrix2_det_entries[R](Nat.0, a), matrix_one[R](Nat.0.suc.suc)),
            fin_zero(Nat.0.suc), fin_succ(Nat.0.suc, fin_zero(Nat.0)))
    matrix_entry_sub[R](Nat.0.suc.suc, Nat.0.suc.suc, square_matrix_mul[R](Nat.0.suc.suc, a, a),
        matrix_smul[R, R](ring_as_module[R], Nat.0.suc.suc, Nat.0.suc.suc, matrix2_trace[R](Nat.0, a), a),
        fin_zero(Nat.0.suc), fin_succ(Nat.0.suc, fin_zero(Nat.0)))
    matrix_entry[R](Nat.0.suc.suc, Nat.0.suc.suc,
        matrix_sub[R](Nat.0.suc.suc, Nat.0.suc.suc, square_matrix_mul[R](Nat.0.suc.suc, a, a),
            matrix_smul[R, R](ring_as_module[R], Nat.0.suc.suc, Nat.0.suc.suc, matrix2_trace[R](Nat.0, a), a)),
        fin_zero(Nat.0.suc), fin_succ(Nat.0.suc, fin_zero(Nat.0))) =
        matrix_entry[R](Nat.0.suc.suc, Nat.0.suc.suc, square_matrix_mul[R](Nat.0.suc.suc, a, a), fin_zero(Nat.0.suc), fin_succ(Nat.0.suc, fin_zero(Nat.0))) -
        matrix_entry[R](Nat.0.suc.suc, Nat.0.suc.suc,
            matrix_smul[R, R](ring_as_module[R], Nat.0.suc.suc, Nat.0.suc.suc, matrix2_trace[R](Nat.0, a), a),
            fin_zero(Nat.0.suc), fin_succ(Nat.0.suc, fin_zero(Nat.0)))
    matrix2_mul_entry_01[R](a, a)
    matrix_entry[R](Nat.0.suc.suc, Nat.0.suc.suc, square_matrix_mul[R](Nat.0.suc.suc, a, a), fin_zero(Nat.0.suc), fin_succ(Nat.0.suc, fin_zero(Nat.0))) =
        matrix2_00[R](Nat.0, a) * matrix2_01[R](Nat.0, a) + matrix2_01[R](Nat.0, a) * matrix2_11[R](Nat.0, a)
    matrix_entry_smul[R, R](ring_as_module[R], Nat.0.suc.suc, Nat.0.suc.suc, matrix2_trace[R](Nat.0, a), a,
        fin_zero(Nat.0.suc), fin_succ(Nat.0.suc, fin_zero(Nat.0)))
    matrix_entry[R](Nat.0.suc.suc, Nat.0.suc.suc,
        matrix_smul[R, R](ring_as_module[R], Nat.0.suc.suc, Nat.0.suc.suc, matrix2_trace[R](Nat.0, a), a),
        fin_zero(Nat.0.suc), fin_succ(Nat.0.suc, fin_zero(Nat.0))) =
        ring_as_module[R].smul(matrix2_trace[R](Nat.0, a), matrix_entry[R](Nat.0.suc.suc, Nat.0.suc.suc, a, fin_zero(Nat.0.suc), fin_succ(Nat.0.suc, fin_zero(Nat.0))))
    ring_as_module_smul[R](matrix2_trace[R](Nat.0, a), matrix_entry[R](Nat.0.suc.suc, Nat.0.suc.suc, a, fin_zero(Nat.0.suc), fin_succ(Nat.0.suc, fin_zero(Nat.0))))
    matrix2_trace[R](Nat.0, a) = matrix2_00[R](Nat.0, a) + matrix2_11[R](Nat.0, a)
    matrix_entry[R](Nat.0.suc.suc, Nat.0.suc.suc,
        matrix_smul[R, R](ring_as_module[R], Nat.0.suc.suc, Nat.0.suc.suc, matrix2_trace[R](Nat.0, a), a),
        fin_zero(Nat.0.suc), fin_succ(Nat.0.suc, fin_zero(Nat.0))) = (matrix2_00[R](Nat.0, a) + matrix2_11[R](Nat.0, a)) * matrix2_01[R](Nat.0, a)
    matrix_entry_smul[R, R](ring_as_module[R], Nat.0.suc.suc, Nat.0.suc.suc, matrix2_det_entries[R](Nat.0, a), matrix_one[R](Nat.0.suc.suc),
        fin_zero(Nat.0.suc), fin_succ(Nat.0.suc, fin_zero(Nat.0)))
    matrix_entry[R](Nat.0.suc.suc, Nat.0.suc.suc,
        matrix_smul[R, R](ring_as_module[R], Nat.0.suc.suc, Nat.0.suc.suc, matrix2_det_entries[R](Nat.0, a), matrix_one[R](Nat.0.suc.suc)),
        fin_zero(Nat.0.suc), fin_succ(Nat.0.suc, fin_zero(Nat.0))) =
        ring_as_module[R].smul(matrix2_det_entries[R](Nat.0, a), matrix_entry[R](Nat.0.suc.suc, Nat.0.suc.suc, matrix_one[R](Nat.0.suc.suc), fin_zero(Nat.0.suc), fin_succ(Nat.0.suc, fin_zero(Nat.0))))
    ring_as_module_smul[R](matrix2_det_entries[R](Nat.0, a), matrix_entry[R](Nat.0.suc.suc, Nat.0.suc.suc, matrix_one[R](Nat.0.suc.suc), fin_zero(Nat.0.suc), fin_succ(Nat.0.suc, fin_zero(Nat.0))))
    matrix_entry_one_off_diag[R](Nat.0.suc.suc, fin_zero(Nat.0.suc), fin_succ(Nat.0.suc, fin_zero(Nat.0)))
    matrix_entry[R](Nat.0.suc.suc, Nat.0.suc.suc, matrix_one[R](Nat.0.suc.suc), fin_zero(Nat.0.suc), fin_succ(Nat.0.suc, fin_zero(Nat.0))) = R.0
    mul_zero_right[R](matrix2_det_entries[R](Nat.0, a))
    matrix_entry[R](Nat.0.suc.suc, Nat.0.suc.suc,
        matrix_smul[R, R](ring_as_module[R], Nat.0.suc.suc, Nat.0.suc.suc, matrix2_det_entries[R](Nat.0, a), matrix_one[R](Nat.0.suc.suc)),
        fin_zero(Nat.0.suc), fin_succ(Nat.0.suc, fin_zero(Nat.0))) = R.0
    matrix_entry[R](Nat.0.suc.suc, Nat.0.suc.suc, cayley_hamilton_lhs[R](Nat.0, a), fin_zero(Nat.0.suc), fin_succ(Nat.0.suc, fin_zero(Nat.0))) =
        (matrix2_00[R](Nat.0, a) * matrix2_01[R](Nat.0, a) + matrix2_01[R](Nat.0, a) * matrix2_11[R](Nat.0, a)) -
            matrix_entry[R](Nat.0.suc.suc, Nat.0.suc.suc,
                matrix_smul[R, R](ring_as_module[R], Nat.0.suc.suc, Nat.0.suc.suc, matrix2_trace[R](Nat.0, a), a),
                fin_zero(Nat.0.suc), fin_succ(Nat.0.suc, fin_zero(Nat.0))) +
            matrix_entry[R](Nat.0.suc.suc, Nat.0.suc.suc,
                matrix_smul[R, R](ring_as_module[R], Nat.0.suc.suc, Nat.0.suc.suc, matrix2_det_entries[R](Nat.0, a), matrix_one[R](Nat.0.suc.suc)),
                fin_zero(Nat.0.suc), fin_succ(Nat.0.suc, fin_zero(Nat.0)))
    matrix_entry[R](Nat.0.suc.suc, Nat.0.suc.suc, cayley_hamilton_lhs[R](Nat.0, a), fin_zero(Nat.0.suc), fin_succ(Nat.0.suc, fin_zero(Nat.0))) =
        (matrix2_00[R](Nat.0, a) * matrix2_01[R](Nat.0, a) + matrix2_01[R](Nat.0, a) * matrix2_11[R](Nat.0, a)) -
            (matrix2_00[R](Nat.0, a) + matrix2_11[R](Nat.0, a)) * matrix2_01[R](Nat.0, a) + R.0
    (matrix2_00[R](Nat.0, a) + matrix2_11[R](Nat.0, a)) * matrix2_01[R](Nat.0, a) =
        matrix2_00[R](Nat.0, a) * matrix2_01[R](Nat.0, a) + matrix2_11[R](Nat.0, a) * matrix2_01[R](Nat.0, a)
    ring_sub_cancel_common[R](matrix2_00[R](Nat.0, a) * matrix2_01[R](Nat.0, a),
        matrix2_01[R](Nat.0, a) * matrix2_11[R](Nat.0, a),
        matrix2_11[R](Nat.0, a) * matrix2_01[R](Nat.0, a))
    (matrix2_00[R](Nat.0, a) * matrix2_01[R](Nat.0, a) + matrix2_01[R](Nat.0, a) * matrix2_11[R](Nat.0, a)) -
        (matrix2_00[R](Nat.0, a) * matrix2_01[R](Nat.0, a) + matrix2_11[R](Nat.0, a) * matrix2_01[R](Nat.0, a)) =
        matrix2_01[R](Nat.0, a) * matrix2_11[R](Nat.0, a) - matrix2_11[R](Nat.0, a) * matrix2_01[R](Nat.0, a)
    matrix_entry[R](Nat.0.suc.suc, Nat.0.suc.suc, cayley_hamilton_lhs[R](Nat.0, a), fin_zero(Nat.0.suc), fin_succ(Nat.0.suc, fin_zero(Nat.0))) =
        (matrix2_01[R](Nat.0, a) * matrix2_11[R](Nat.0, a) - matrix2_11[R](Nat.0, a) * matrix2_01[R](Nat.0, a)) + R.0
    ring_mul_comm_sub_zero[R](matrix2_01[R](Nat.0, a), matrix2_11[R](Nat.0, a))
    matrix2_01[R](Nat.0, a) * matrix2_11[R](Nat.0, a) - matrix2_11[R](Nat.0, a) * matrix2_01[R](Nat.0, a) = R.0
    (matrix2_01[R](Nat.0, a) * matrix2_11[R](Nat.0, a) - matrix2_11[R](Nat.0, a) * matrix2_01[R](Nat.0, a)) + R.0 = R.0
    matrix_entry[R](Nat.0.suc.suc, Nat.0.suc.suc, cayley_hamilton_lhs[R](Nat.0, a), fin_zero(Nat.0.suc), fin_succ(Nat.0.suc, fin_zero(Nat.0))) = R.0
}

/// The lower-left entry of the two-by-two Cayley-Hamilton expression is zero.
theorem cayley_hamilton_entry_10[R: CommRing](a: Matrix[R, Nat.0.suc.suc, Nat.0.suc.suc]) {
    matrix_entry[R](Nat.0.suc.suc, Nat.0.suc.suc, cayley_hamilton_lhs[R](Nat.0, a),
        fin_succ(Nat.0.suc, fin_zero(Nat.0)), fin_zero(Nat.0.suc)) = R.0
} by {
    matrix_entry[R](Nat.0.suc.suc, Nat.0.suc.suc, cayley_hamilton_lhs[R](Nat.0, a), fin_succ(Nat.0.suc, fin_zero(Nat.0)), fin_zero(Nat.0.suc)) =
        matrix_entry[R](Nat.0.suc.suc, Nat.0.suc.suc,
            matrix_add[R](Nat.0.suc.suc, Nat.0.suc.suc,
                matrix_sub[R](Nat.0.suc.suc, Nat.0.suc.suc, square_matrix_mul[R](Nat.0.suc.suc, a, a),
                    matrix_smul[R, R](ring_as_module[R], Nat.0.suc.suc, Nat.0.suc.suc, matrix2_trace[R](Nat.0, a), a)),
                matrix_smul[R, R](ring_as_module[R], Nat.0.suc.suc, Nat.0.suc.suc, matrix2_det_entries[R](Nat.0, a), matrix_one[R](Nat.0.suc.suc))),
            fin_succ(Nat.0.suc, fin_zero(Nat.0)), fin_zero(Nat.0.suc))
    matrix_entry_add[R](Nat.0.suc.suc, Nat.0.suc.suc,
        matrix_sub[R](Nat.0.suc.suc, Nat.0.suc.suc, square_matrix_mul[R](Nat.0.suc.suc, a, a),
            matrix_smul[R, R](ring_as_module[R], Nat.0.suc.suc, Nat.0.suc.suc, matrix2_trace[R](Nat.0, a), a)),
        matrix_smul[R, R](ring_as_module[R], Nat.0.suc.suc, Nat.0.suc.suc, matrix2_det_entries[R](Nat.0, a), matrix_one[R](Nat.0.suc.suc)),
        fin_succ(Nat.0.suc, fin_zero(Nat.0)), fin_zero(Nat.0.suc))
    matrix_entry[R](Nat.0.suc.suc, Nat.0.suc.suc,
        matrix_add[R](Nat.0.suc.suc, Nat.0.suc.suc,
            matrix_sub[R](Nat.0.suc.suc, Nat.0.suc.suc, square_matrix_mul[R](Nat.0.suc.suc, a, a),
                matrix_smul[R, R](ring_as_module[R], Nat.0.suc.suc, Nat.0.suc.suc, matrix2_trace[R](Nat.0, a), a)),
            matrix_smul[R, R](ring_as_module[R], Nat.0.suc.suc, Nat.0.suc.suc, matrix2_det_entries[R](Nat.0, a), matrix_one[R](Nat.0.suc.suc))),
        fin_succ(Nat.0.suc, fin_zero(Nat.0)), fin_zero(Nat.0.suc)) =
        matrix_entry[R](Nat.0.suc.suc, Nat.0.suc.suc,
            matrix_sub[R](Nat.0.suc.suc, Nat.0.suc.suc, square_matrix_mul[R](Nat.0.suc.suc, a, a),
                matrix_smul[R, R](ring_as_module[R], Nat.0.suc.suc, Nat.0.suc.suc, matrix2_trace[R](Nat.0, a), a)),
            fin_succ(Nat.0.suc, fin_zero(Nat.0)), fin_zero(Nat.0.suc)) +
        matrix_entry[R](Nat.0.suc.suc, Nat.0.suc.suc,
            matrix_smul[R, R](ring_as_module[R], Nat.0.suc.suc, Nat.0.suc.suc, matrix2_det_entries[R](Nat.0, a), matrix_one[R](Nat.0.suc.suc)),
            fin_succ(Nat.0.suc, fin_zero(Nat.0)), fin_zero(Nat.0.suc))
    matrix_entry_sub[R](Nat.0.suc.suc, Nat.0.suc.suc, square_matrix_mul[R](Nat.0.suc.suc, a, a),
        matrix_smul[R, R](ring_as_module[R], Nat.0.suc.suc, Nat.0.suc.suc, matrix2_trace[R](Nat.0, a), a),
        fin_succ(Nat.0.suc, fin_zero(Nat.0)), fin_zero(Nat.0.suc))
    matrix_entry[R](Nat.0.suc.suc, Nat.0.suc.suc,
        matrix_sub[R](Nat.0.suc.suc, Nat.0.suc.suc, square_matrix_mul[R](Nat.0.suc.suc, a, a),
            matrix_smul[R, R](ring_as_module[R], Nat.0.suc.suc, Nat.0.suc.suc, matrix2_trace[R](Nat.0, a), a)),
        fin_succ(Nat.0.suc, fin_zero(Nat.0)), fin_zero(Nat.0.suc)) =
        matrix_entry[R](Nat.0.suc.suc, Nat.0.suc.suc, square_matrix_mul[R](Nat.0.suc.suc, a, a), fin_succ(Nat.0.suc, fin_zero(Nat.0)), fin_zero(Nat.0.suc)) -
        matrix_entry[R](Nat.0.suc.suc, Nat.0.suc.suc,
            matrix_smul[R, R](ring_as_module[R], Nat.0.suc.suc, Nat.0.suc.suc, matrix2_trace[R](Nat.0, a), a),
            fin_succ(Nat.0.suc, fin_zero(Nat.0)), fin_zero(Nat.0.suc))
    matrix2_mul_entry_10[R](a, a)
    matrix_entry[R](Nat.0.suc.suc, Nat.0.suc.suc, square_matrix_mul[R](Nat.0.suc.suc, a, a), fin_succ(Nat.0.suc, fin_zero(Nat.0)), fin_zero(Nat.0.suc)) =
        matrix2_10[R](Nat.0, a) * matrix2_00[R](Nat.0, a) + matrix2_11[R](Nat.0, a) * matrix2_10[R](Nat.0, a)
    matrix_entry_smul[R, R](ring_as_module[R], Nat.0.suc.suc, Nat.0.suc.suc, matrix2_trace[R](Nat.0, a), a,
        fin_succ(Nat.0.suc, fin_zero(Nat.0)), fin_zero(Nat.0.suc))
    matrix_entry[R](Nat.0.suc.suc, Nat.0.suc.suc,
        matrix_smul[R, R](ring_as_module[R], Nat.0.suc.suc, Nat.0.suc.suc, matrix2_trace[R](Nat.0, a), a),
        fin_succ(Nat.0.suc, fin_zero(Nat.0)), fin_zero(Nat.0.suc)) =
        ring_as_module[R].smul(matrix2_trace[R](Nat.0, a), matrix_entry[R](Nat.0.suc.suc, Nat.0.suc.suc, a, fin_succ(Nat.0.suc, fin_zero(Nat.0)), fin_zero(Nat.0.suc)))
    ring_as_module_smul[R](matrix2_trace[R](Nat.0, a), matrix_entry[R](Nat.0.suc.suc, Nat.0.suc.suc, a, fin_succ(Nat.0.suc, fin_zero(Nat.0)), fin_zero(Nat.0.suc)))
    matrix2_trace[R](Nat.0, a) = matrix2_00[R](Nat.0, a) + matrix2_11[R](Nat.0, a)
    matrix_entry[R](Nat.0.suc.suc, Nat.0.suc.suc,
        matrix_smul[R, R](ring_as_module[R], Nat.0.suc.suc, Nat.0.suc.suc, matrix2_trace[R](Nat.0, a), a),
        fin_succ(Nat.0.suc, fin_zero(Nat.0)), fin_zero(Nat.0.suc)) = (matrix2_00[R](Nat.0, a) + matrix2_11[R](Nat.0, a)) * matrix2_10[R](Nat.0, a)
    matrix_entry_smul[R, R](ring_as_module[R], Nat.0.suc.suc, Nat.0.suc.suc, matrix2_det_entries[R](Nat.0, a), matrix_one[R](Nat.0.suc.suc),
        fin_succ(Nat.0.suc, fin_zero(Nat.0)), fin_zero(Nat.0.suc))
    matrix_entry[R](Nat.0.suc.suc, Nat.0.suc.suc,
        matrix_smul[R, R](ring_as_module[R], Nat.0.suc.suc, Nat.0.suc.suc, matrix2_det_entries[R](Nat.0, a), matrix_one[R](Nat.0.suc.suc)),
        fin_succ(Nat.0.suc, fin_zero(Nat.0)), fin_zero(Nat.0.suc)) =
        ring_as_module[R].smul(matrix2_det_entries[R](Nat.0, a), matrix_entry[R](Nat.0.suc.suc, Nat.0.suc.suc, matrix_one[R](Nat.0.suc.suc), fin_succ(Nat.0.suc, fin_zero(Nat.0)), fin_zero(Nat.0.suc)))
    ring_as_module_smul[R](matrix2_det_entries[R](Nat.0, a), matrix_entry[R](Nat.0.suc.suc, Nat.0.suc.suc, matrix_one[R](Nat.0.suc.suc), fin_succ(Nat.0.suc, fin_zero(Nat.0)), fin_zero(Nat.0.suc)))
    matrix_entry_one_off_diag[R](Nat.0.suc.suc, fin_succ(Nat.0.suc, fin_zero(Nat.0)), fin_zero(Nat.0.suc))
    matrix_entry[R](Nat.0.suc.suc, Nat.0.suc.suc, matrix_one[R](Nat.0.suc.suc), fin_succ(Nat.0.suc, fin_zero(Nat.0)), fin_zero(Nat.0.suc)) = R.0
    mul_zero_right[R](matrix2_det_entries[R](Nat.0, a))
    matrix_entry[R](Nat.0.suc.suc, Nat.0.suc.suc,
        matrix_smul[R, R](ring_as_module[R], Nat.0.suc.suc, Nat.0.suc.suc, matrix2_det_entries[R](Nat.0, a), matrix_one[R](Nat.0.suc.suc)),
        fin_succ(Nat.0.suc, fin_zero(Nat.0)), fin_zero(Nat.0.suc)) = R.0
    matrix_entry[R](Nat.0.suc.suc, Nat.0.suc.suc, cayley_hamilton_lhs[R](Nat.0, a), fin_succ(Nat.0.suc, fin_zero(Nat.0)), fin_zero(Nat.0.suc)) =
        (matrix2_10[R](Nat.0, a) * matrix2_00[R](Nat.0, a) + matrix2_11[R](Nat.0, a) * matrix2_10[R](Nat.0, a)) -
            matrix_entry[R](Nat.0.suc.suc, Nat.0.suc.suc,
                matrix_smul[R, R](ring_as_module[R], Nat.0.suc.suc, Nat.0.suc.suc, matrix2_trace[R](Nat.0, a), a),
                fin_succ(Nat.0.suc, fin_zero(Nat.0)), fin_zero(Nat.0.suc)) +
            matrix_entry[R](Nat.0.suc.suc, Nat.0.suc.suc,
                matrix_smul[R, R](ring_as_module[R], Nat.0.suc.suc, Nat.0.suc.suc, matrix2_det_entries[R](Nat.0, a), matrix_one[R](Nat.0.suc.suc)),
                fin_succ(Nat.0.suc, fin_zero(Nat.0)), fin_zero(Nat.0.suc))
    matrix_entry[R](Nat.0.suc.suc, Nat.0.suc.suc, cayley_hamilton_lhs[R](Nat.0, a), fin_succ(Nat.0.suc, fin_zero(Nat.0)), fin_zero(Nat.0.suc)) =
        (matrix2_10[R](Nat.0, a) * matrix2_00[R](Nat.0, a) + matrix2_11[R](Nat.0, a) * matrix2_10[R](Nat.0, a)) -
            (matrix2_00[R](Nat.0, a) + matrix2_11[R](Nat.0, a)) * matrix2_10[R](Nat.0, a) + R.0
    (matrix2_00[R](Nat.0, a) + matrix2_11[R](Nat.0, a)) * matrix2_10[R](Nat.0, a) =
        matrix2_00[R](Nat.0, a) * matrix2_10[R](Nat.0, a) + matrix2_11[R](Nat.0, a) * matrix2_10[R](Nat.0, a)
    matrix2_10[R](Nat.0, a) * matrix2_00[R](Nat.0, a) = matrix2_00[R](Nat.0, a) * matrix2_10[R](Nat.0, a)
    (matrix2_10[R](Nat.0, a) * matrix2_00[R](Nat.0, a) + matrix2_11[R](Nat.0, a) * matrix2_10[R](Nat.0, a)) -
        (matrix2_00[R](Nat.0, a) * matrix2_10[R](Nat.0, a) + matrix2_11[R](Nat.0, a) * matrix2_10[R](Nat.0, a)) = R.0
    matrix_entry[R](Nat.0.suc.suc, Nat.0.suc.suc, cayley_hamilton_lhs[R](Nat.0, a), fin_succ(Nat.0.suc, fin_zero(Nat.0)), fin_zero(Nat.0.suc)) = R.0 + R.0
    matrix_entry[R](Nat.0.suc.suc, Nat.0.suc.suc, cayley_hamilton_lhs[R](Nat.0, a), fin_succ(Nat.0.suc, fin_zero(Nat.0)), fin_zero(Nat.0.suc)) = R.0
}

/// The lower-right entry of the two-by-two Cayley-Hamilton expression is zero.
theorem cayley_hamilton_entry_11[R: CommRing](a: Matrix[R, Nat.0.suc.suc, Nat.0.suc.suc]) {
    matrix_entry[R](Nat.0.suc.suc, Nat.0.suc.suc, cayley_hamilton_lhs[R](Nat.0, a),
        fin_succ(Nat.0.suc, fin_zero(Nat.0)), fin_succ(Nat.0.suc, fin_zero(Nat.0))) = R.0
} by {
    matrix_entry[R](Nat.0.suc.suc, Nat.0.suc.suc, cayley_hamilton_lhs[R](Nat.0, a), fin_succ(Nat.0.suc, fin_zero(Nat.0)), fin_succ(Nat.0.suc, fin_zero(Nat.0))) =
        matrix_entry[R](Nat.0.suc.suc, Nat.0.suc.suc,
            matrix_add[R](Nat.0.suc.suc, Nat.0.suc.suc,
                matrix_sub[R](Nat.0.suc.suc, Nat.0.suc.suc, square_matrix_mul[R](Nat.0.suc.suc, a, a),
                    matrix_smul[R, R](ring_as_module[R], Nat.0.suc.suc, Nat.0.suc.suc, matrix2_trace[R](Nat.0, a), a)),
                matrix_smul[R, R](ring_as_module[R], Nat.0.suc.suc, Nat.0.suc.suc, matrix2_det_entries[R](Nat.0, a), matrix_one[R](Nat.0.suc.suc))),
            fin_succ(Nat.0.suc, fin_zero(Nat.0)), fin_succ(Nat.0.suc, fin_zero(Nat.0)))
    matrix_entry_add[R](Nat.0.suc.suc, Nat.0.suc.suc,
        matrix_sub[R](Nat.0.suc.suc, Nat.0.suc.suc, square_matrix_mul[R](Nat.0.suc.suc, a, a),
            matrix_smul[R, R](ring_as_module[R], Nat.0.suc.suc, Nat.0.suc.suc, matrix2_trace[R](Nat.0, a), a)),
        matrix_smul[R, R](ring_as_module[R], Nat.0.suc.suc, Nat.0.suc.suc, matrix2_det_entries[R](Nat.0, a), matrix_one[R](Nat.0.suc.suc)),
        fin_succ(Nat.0.suc, fin_zero(Nat.0)), fin_succ(Nat.0.suc, fin_zero(Nat.0)))
    matrix_entry[R](Nat.0.suc.suc, Nat.0.suc.suc,
        matrix_add[R](Nat.0.suc.suc, Nat.0.suc.suc,
            matrix_sub[R](Nat.0.suc.suc, Nat.0.suc.suc, square_matrix_mul[R](Nat.0.suc.suc, a, a),
                matrix_smul[R, R](ring_as_module[R], Nat.0.suc.suc, Nat.0.suc.suc, matrix2_trace[R](Nat.0, a), a)),
            matrix_smul[R, R](ring_as_module[R], Nat.0.suc.suc, Nat.0.suc.suc, matrix2_det_entries[R](Nat.0, a), matrix_one[R](Nat.0.suc.suc))),
        fin_succ(Nat.0.suc, fin_zero(Nat.0)), fin_succ(Nat.0.suc, fin_zero(Nat.0))) =
        matrix_entry[R](Nat.0.suc.suc, Nat.0.suc.suc,
            matrix_sub[R](Nat.0.suc.suc, Nat.0.suc.suc, square_matrix_mul[R](Nat.0.suc.suc, a, a),
                matrix_smul[R, R](ring_as_module[R], Nat.0.suc.suc, Nat.0.suc.suc, matrix2_trace[R](Nat.0, a), a)),
            fin_succ(Nat.0.suc, fin_zero(Nat.0)), fin_succ(Nat.0.suc, fin_zero(Nat.0))) +
        matrix_entry[R](Nat.0.suc.suc, Nat.0.suc.suc,
            matrix_smul[R, R](ring_as_module[R], Nat.0.suc.suc, Nat.0.suc.suc, matrix2_det_entries[R](Nat.0, a), matrix_one[R](Nat.0.suc.suc)),
            fin_succ(Nat.0.suc, fin_zero(Nat.0)), fin_succ(Nat.0.suc, fin_zero(Nat.0)))
    matrix_entry_sub[R](Nat.0.suc.suc, Nat.0.suc.suc, square_matrix_mul[R](Nat.0.suc.suc, a, a),
        matrix_smul[R, R](ring_as_module[R], Nat.0.suc.suc, Nat.0.suc.suc, matrix2_trace[R](Nat.0, a), a),
        fin_succ(Nat.0.suc, fin_zero(Nat.0)), fin_succ(Nat.0.suc, fin_zero(Nat.0)))
    matrix_entry[R](Nat.0.suc.suc, Nat.0.suc.suc,
        matrix_sub[R](Nat.0.suc.suc, Nat.0.suc.suc, square_matrix_mul[R](Nat.0.suc.suc, a, a),
            matrix_smul[R, R](ring_as_module[R], Nat.0.suc.suc, Nat.0.suc.suc, matrix2_trace[R](Nat.0, a), a)),
        fin_succ(Nat.0.suc, fin_zero(Nat.0)), fin_succ(Nat.0.suc, fin_zero(Nat.0))) =
        matrix_entry[R](Nat.0.suc.suc, Nat.0.suc.suc, square_matrix_mul[R](Nat.0.suc.suc, a, a), fin_succ(Nat.0.suc, fin_zero(Nat.0)), fin_succ(Nat.0.suc, fin_zero(Nat.0))) -
        matrix_entry[R](Nat.0.suc.suc, Nat.0.suc.suc,
            matrix_smul[R, R](ring_as_module[R], Nat.0.suc.suc, Nat.0.suc.suc, matrix2_trace[R](Nat.0, a), a),
            fin_succ(Nat.0.suc, fin_zero(Nat.0)), fin_succ(Nat.0.suc, fin_zero(Nat.0)))
    matrix2_mul_entry_11[R](a, a)
    matrix_entry[R](Nat.0.suc.suc, Nat.0.suc.suc, square_matrix_mul[R](Nat.0.suc.suc, a, a), fin_succ(Nat.0.suc, fin_zero(Nat.0)), fin_succ(Nat.0.suc, fin_zero(Nat.0))) =
        matrix2_10[R](Nat.0, a) * matrix2_01[R](Nat.0, a) + matrix2_11[R](Nat.0, a) * matrix2_11[R](Nat.0, a)
    matrix_entry_smul[R, R](ring_as_module[R], Nat.0.suc.suc, Nat.0.suc.suc, matrix2_trace[R](Nat.0, a), a,
        fin_succ(Nat.0.suc, fin_zero(Nat.0)), fin_succ(Nat.0.suc, fin_zero(Nat.0)))
    matrix_entry[R](Nat.0.suc.suc, Nat.0.suc.suc,
        matrix_smul[R, R](ring_as_module[R], Nat.0.suc.suc, Nat.0.suc.suc, matrix2_trace[R](Nat.0, a), a),
        fin_succ(Nat.0.suc, fin_zero(Nat.0)), fin_succ(Nat.0.suc, fin_zero(Nat.0))) =
        ring_as_module[R].smul(matrix2_trace[R](Nat.0, a), matrix_entry[R](Nat.0.suc.suc, Nat.0.suc.suc, a, fin_succ(Nat.0.suc, fin_zero(Nat.0)), fin_succ(Nat.0.suc, fin_zero(Nat.0))))
    ring_as_module_smul[R](matrix2_trace[R](Nat.0, a), matrix_entry[R](Nat.0.suc.suc, Nat.0.suc.suc, a, fin_succ(Nat.0.suc, fin_zero(Nat.0)), fin_succ(Nat.0.suc, fin_zero(Nat.0))))
    matrix2_trace[R](Nat.0, a) = matrix2_00[R](Nat.0, a) + matrix2_11[R](Nat.0, a)
    matrix_entry[R](Nat.0.suc.suc, Nat.0.suc.suc,
        matrix_smul[R, R](ring_as_module[R], Nat.0.suc.suc, Nat.0.suc.suc, matrix2_trace[R](Nat.0, a), a),
        fin_succ(Nat.0.suc, fin_zero(Nat.0)), fin_succ(Nat.0.suc, fin_zero(Nat.0))) = (matrix2_00[R](Nat.0, a) + matrix2_11[R](Nat.0, a)) * matrix2_11[R](Nat.0, a)
    matrix_entry_smul[R, R](ring_as_module[R], Nat.0.suc.suc, Nat.0.suc.suc, matrix2_det_entries[R](Nat.0, a), matrix_one[R](Nat.0.suc.suc),
        fin_succ(Nat.0.suc, fin_zero(Nat.0)), fin_succ(Nat.0.suc, fin_zero(Nat.0)))
    matrix_entry[R](Nat.0.suc.suc, Nat.0.suc.suc,
        matrix_smul[R, R](ring_as_module[R], Nat.0.suc.suc, Nat.0.suc.suc, matrix2_det_entries[R](Nat.0, a), matrix_one[R](Nat.0.suc.suc)),
        fin_succ(Nat.0.suc, fin_zero(Nat.0)), fin_succ(Nat.0.suc, fin_zero(Nat.0))) =
        ring_as_module[R].smul(matrix2_det_entries[R](Nat.0, a), matrix_entry[R](Nat.0.suc.suc, Nat.0.suc.suc, matrix_one[R](Nat.0.suc.suc), fin_succ(Nat.0.suc, fin_zero(Nat.0)), fin_succ(Nat.0.suc, fin_zero(Nat.0))))
    ring_as_module_smul[R](matrix2_det_entries[R](Nat.0, a), matrix_entry[R](Nat.0.suc.suc, Nat.0.suc.suc, matrix_one[R](Nat.0.suc.suc), fin_succ(Nat.0.suc, fin_zero(Nat.0)), fin_succ(Nat.0.suc, fin_zero(Nat.0))))
    matrix_entry_one_diag[R](Nat.0.suc.suc, fin_succ(Nat.0.suc, fin_zero(Nat.0)))
    matrix_entry[R](Nat.0.suc.suc, Nat.0.suc.suc, matrix_one[R](Nat.0.suc.suc), fin_succ(Nat.0.suc, fin_zero(Nat.0)), fin_succ(Nat.0.suc, fin_zero(Nat.0))) = R.1
    matrix2_det_entries[R](Nat.0, a) =
        matrix2_00[R](Nat.0, a) * matrix2_11[R](Nat.0, a) - matrix2_01[R](Nat.0, a) * matrix2_10[R](Nat.0, a)
    matrix_entry[R](Nat.0.suc.suc, Nat.0.suc.suc, cayley_hamilton_lhs[R](Nat.0, a), fin_succ(Nat.0.suc, fin_zero(Nat.0)), fin_succ(Nat.0.suc, fin_zero(Nat.0))) =
        (matrix2_10[R](Nat.0, a) * matrix2_01[R](Nat.0, a) + matrix2_11[R](Nat.0, a) * matrix2_11[R](Nat.0, a)) -
            matrix_entry[R](Nat.0.suc.suc, Nat.0.suc.suc,
                matrix_smul[R, R](ring_as_module[R], Nat.0.suc.suc, Nat.0.suc.suc, matrix2_trace[R](Nat.0, a), a),
                fin_succ(Nat.0.suc, fin_zero(Nat.0)), fin_succ(Nat.0.suc, fin_zero(Nat.0))) +
            matrix_entry[R](Nat.0.suc.suc, Nat.0.suc.suc,
                matrix_smul[R, R](ring_as_module[R], Nat.0.suc.suc, Nat.0.suc.suc, matrix2_det_entries[R](Nat.0, a), matrix_one[R](Nat.0.suc.suc)),
                fin_succ(Nat.0.suc, fin_zero(Nat.0)), fin_succ(Nat.0.suc, fin_zero(Nat.0)))
    matrix_entry[R](Nat.0.suc.suc, Nat.0.suc.suc, cayley_hamilton_lhs[R](Nat.0, a), fin_succ(Nat.0.suc, fin_zero(Nat.0)), fin_succ(Nat.0.suc, fin_zero(Nat.0))) =
        (matrix2_10[R](Nat.0, a) * matrix2_01[R](Nat.0, a) + matrix2_11[R](Nat.0, a) * matrix2_11[R](Nat.0, a)) -
            (matrix2_00[R](Nat.0, a) + matrix2_11[R](Nat.0, a)) * matrix2_11[R](Nat.0, a) +
            (matrix2_00[R](Nat.0, a) * matrix2_11[R](Nat.0, a) - matrix2_01[R](Nat.0, a) * matrix2_10[R](Nat.0, a))
    (matrix2_00[R](Nat.0, a) + matrix2_11[R](Nat.0, a)) * matrix2_11[R](Nat.0, a) =
        matrix2_00[R](Nat.0, a) * matrix2_11[R](Nat.0, a) + matrix2_11[R](Nat.0, a) * matrix2_11[R](Nat.0, a)
    matrix2_10[R](Nat.0, a) * matrix2_01[R](Nat.0, a) = matrix2_01[R](Nat.0, a) * matrix2_10[R](Nat.0, a)
    ring_sub_cancel_common[R](matrix2_11[R](Nat.0, a) * matrix2_11[R](Nat.0, a),
        matrix2_01[R](Nat.0, a) * matrix2_10[R](Nat.0, a),
        matrix2_00[R](Nat.0, a) * matrix2_11[R](Nat.0, a))
    (matrix2_11[R](Nat.0, a) * matrix2_11[R](Nat.0, a) + matrix2_01[R](Nat.0, a) * matrix2_10[R](Nat.0, a)) -
        (matrix2_11[R](Nat.0, a) * matrix2_11[R](Nat.0, a) + matrix2_00[R](Nat.0, a) * matrix2_11[R](Nat.0, a)) =
        matrix2_01[R](Nat.0, a) * matrix2_10[R](Nat.0, a) - matrix2_00[R](Nat.0, a) * matrix2_11[R](Nat.0, a)
    matrix_entry[R](Nat.0.suc.suc, Nat.0.suc.suc, cayley_hamilton_lhs[R](Nat.0, a), fin_succ(Nat.0.suc, fin_zero(Nat.0)), fin_succ(Nat.0.suc, fin_zero(Nat.0))) =
        (matrix2_01[R](Nat.0, a) * matrix2_10[R](Nat.0, a) - matrix2_00[R](Nat.0, a) * matrix2_11[R](Nat.0, a)) +
            (matrix2_00[R](Nat.0, a) * matrix2_11[R](Nat.0, a) - matrix2_01[R](Nat.0, a) * matrix2_10[R](Nat.0, a))
    ring_add_sub_cancel[R](matrix2_01[R](Nat.0, a) * matrix2_10[R](Nat.0, a),
        matrix2_00[R](Nat.0, a) * matrix2_11[R](Nat.0, a),
        matrix2_00[R](Nat.0, a) * matrix2_11[R](Nat.0, a))
    (matrix2_01[R](Nat.0, a) * matrix2_10[R](Nat.0, a) - matrix2_00[R](Nat.0, a) * matrix2_11[R](Nat.0, a)) +
        (matrix2_00[R](Nat.0, a) * matrix2_11[R](Nat.0, a) - matrix2_01[R](Nat.0, a) * matrix2_10[R](Nat.0, a)) =
        matrix2_00[R](Nat.0, a) * matrix2_11[R](Nat.0, a) - matrix2_00[R](Nat.0, a) * matrix2_11[R](Nat.0, a)
    matrix2_00[R](Nat.0, a) * matrix2_11[R](Nat.0, a) - matrix2_00[R](Nat.0, a) * matrix2_11[R](Nat.0, a) = R.0
    matrix_entry[R](Nat.0.suc.suc, Nat.0.suc.suc, cayley_hamilton_lhs[R](Nat.0, a), fin_succ(Nat.0.suc, fin_zero(Nat.0)), fin_succ(Nat.0.suc, fin_zero(Nat.0))) = R.0
}

/// Every entry of the two-by-two Cayley-Hamilton expression is zero.
theorem cayley_hamilton_two_entries[R: CommRing](a: Matrix[R, Nat.0.suc.suc, Nat.0.suc.suc]) {
    forall(i: Fin[Nat.0.suc.suc], j: Fin[Nat.0.suc.suc]) {
        matrix_entry[R](Nat.0.suc.suc, Nat.0.suc.suc, cayley_hamilton_lhs[R](Nat.0, a), i, j) = R.0
    }
} by {
    forall(i: Fin[Nat.0.suc.suc], j: Fin[Nat.0.suc.suc]) {
        fin_two_index_cases(i)
        fin_two_index_cases(j)
        if i.value = Nat.0 {
            if j.value = Nat.0 {
                fin_two_zero_value_two
                i.value = fin_zero(Nat.0.suc).value
                ext(Nat.0.suc.suc, i, fin_zero(Nat.0.suc))
                i = fin_zero(Nat.0.suc)
                fin_two_zero_value_two
                j.value = fin_zero(Nat.0.suc).value
                ext(Nat.0.suc.suc, j, fin_zero(Nat.0.suc))
                j = fin_zero(Nat.0.suc)
                cayley_hamilton_entry_00[R](a)
                matrix_entry[R](Nat.0.suc.suc, Nat.0.suc.suc, cayley_hamilton_lhs[R](Nat.0, a), fin_zero(Nat.0.suc), fin_zero(Nat.0.suc)) = R.0
                matrix_entry[R](Nat.0.suc.suc, Nat.0.suc.suc, cayley_hamilton_lhs[R](Nat.0, a), i, j) = R.0
            } else {
                fin_two_zero_value_two
                i.value = fin_zero(Nat.0.suc).value
                ext(Nat.0.suc.suc, i, fin_zero(Nat.0.suc))
                i = fin_zero(Nat.0.suc)
                fin_two_one_value_two
                j.value = fin_succ(Nat.0.suc, fin_zero(Nat.0)).value
                ext(Nat.0.suc.suc, j, fin_succ(Nat.0.suc, fin_zero(Nat.0)))
                j = fin_succ(Nat.0.suc, fin_zero(Nat.0))
                cayley_hamilton_entry_01[R](a)
                matrix_entry[R](Nat.0.suc.suc, Nat.0.suc.suc, cayley_hamilton_lhs[R](Nat.0, a), fin_zero(Nat.0.suc), fin_succ(Nat.0.suc, fin_zero(Nat.0))) = R.0
                matrix_entry[R](Nat.0.suc.suc, Nat.0.suc.suc, cayley_hamilton_lhs[R](Nat.0, a), i, j) = R.0
            }
        } else {
            if j.value = Nat.0 {
                fin_two_one_value_two
                i.value = fin_succ(Nat.0.suc, fin_zero(Nat.0)).value
                ext(Nat.0.suc.suc, i, fin_succ(Nat.0.suc, fin_zero(Nat.0)))
                i = fin_succ(Nat.0.suc, fin_zero(Nat.0))
                fin_two_zero_value_two
                j.value = fin_zero(Nat.0.suc).value
                ext(Nat.0.suc.suc, j, fin_zero(Nat.0.suc))
                j = fin_zero(Nat.0.suc)
                cayley_hamilton_entry_10[R](a)
                matrix_entry[R](Nat.0.suc.suc, Nat.0.suc.suc, cayley_hamilton_lhs[R](Nat.0, a), fin_succ(Nat.0.suc, fin_zero(Nat.0)), fin_zero(Nat.0.suc)) = R.0
                matrix_entry[R](Nat.0.suc.suc, Nat.0.suc.suc, cayley_hamilton_lhs[R](Nat.0, a), i, j) = R.0
            } else {
                fin_two_one_value_two
                i.value = fin_succ(Nat.0.suc, fin_zero(Nat.0)).value
                ext(Nat.0.suc.suc, i, fin_succ(Nat.0.suc, fin_zero(Nat.0)))
                i = fin_succ(Nat.0.suc, fin_zero(Nat.0))
                fin_two_one_value_two
                j.value = fin_succ(Nat.0.suc, fin_zero(Nat.0)).value
                ext(Nat.0.suc.suc, j, fin_succ(Nat.0.suc, fin_zero(Nat.0)))
                j = fin_succ(Nat.0.suc, fin_zero(Nat.0))
                cayley_hamilton_entry_11[R](a)
                matrix_entry[R](Nat.0.suc.suc, Nat.0.suc.suc, cayley_hamilton_lhs[R](Nat.0, a), fin_succ(Nat.0.suc, fin_zero(Nat.0)), fin_succ(Nat.0.suc, fin_zero(Nat.0))) = R.0
                matrix_entry[R](Nat.0.suc.suc, Nat.0.suc.suc, cayley_hamilton_lhs[R](Nat.0, a), i, j) = R.0
            }
        }
    }
}

/// The two-by-two Cayley-Hamilton identity: substituting the characteristic
/// polynomial into the matrix gives the zero matrix.
theorem cayley_hamilton_two[R: CommRing](a: Matrix[R, Nat.0.suc.suc, Nat.0.suc.suc]) {
    cayley_hamilton_lhs[R](Nat.0, a) = matrix_zero[R](Nat.0.suc.suc, Nat.0.suc.suc)
} by {
    cayley_hamilton_two_entries[R](a)
    forall(i: Fin[Nat.0.suc.suc], j: Fin[Nat.0.suc.suc]) {
        matrix_entry[R](Nat.0.suc.suc, Nat.0.suc.suc, cayley_hamilton_lhs[R](Nat.0, a), i, j) = R.0
        matrix_entry_zero[R](Nat.0.suc.suc, Nat.0.suc.suc, i, j)
        matrix_entry[R](Nat.0.suc.suc, Nat.0.suc.suc, matrix_zero[R](Nat.0.suc.suc, Nat.0.suc.suc), i, j) = R.0
        matrix_entry[R](Nat.0.suc.suc, Nat.0.suc.suc, cayley_hamilton_lhs[R](Nat.0, a), i, j) =
            matrix_entry[R](Nat.0.suc.suc, Nat.0.suc.suc, matrix_zero[R](Nat.0.suc.suc, Nat.0.suc.suc), i, j)
    }
    matrix_ext[R](Nat.0.suc.suc, Nat.0.suc.suc, cayley_hamilton_lhs[R](Nat.0, a), matrix_zero[R](Nat.0.suc.suc, Nat.0.suc.suc))
    cayley_hamilton_lhs[R](Nat.0, a) = matrix_zero[R](Nat.0.suc.suc, Nat.0.suc.suc)
}

// ---------------------------------------------------------------------------
// Eigenvalues and the characteristic polynomial
// ---------------------------------------------------------------------------

/// Moving a negated trailing summand to the back of an addition.
theorem ring_move_neg_to_back[R: Ring](x: R, y: R, z: R) {
    (x - z) + y = x + y - z
} by {
    (x - z) + y = (x + -z) + (y + R.0)
    add_swap_inner[R](x, -z, y, R.0)
    (x + -z) + (y + R.0) = (x + y) + (-z + R.0)
    (x + y) + (-z + R.0) = x + y - z
    (x - z) + y = x + y - z
}

/// Applying `A - lam * I` to a vector at the first coordinate gives the
/// application of `A` minus `lam` times the coordinate.
theorem matrix_apply_char_poly_sub_0[R: CommRing](
    a: Matrix[R, Nat.0.suc.suc, Nat.0.suc.suc], lam: R, v: Fin[Nat.0.suc.suc] -> R
) {
    matrix_apply[R](Nat.0.suc.suc,
        matrix_sub[R](Nat.0.suc.suc, Nat.0.suc.suc, a,
            matrix_smul[R, R](ring_as_module[R], Nat.0.suc.suc, Nat.0.suc.suc, lam, matrix_one[R](Nat.0.suc.suc))),
        v, fin_zero(Nat.0.suc)) =
        matrix_apply[R](Nat.0.suc.suc, a, v, fin_zero(Nat.0.suc)) - lam * v(fin_zero(Nat.0.suc))
} by {
    matrix_apply_eq[R](Nat.0.suc.suc,
        matrix_sub[R](Nat.0.suc.suc, Nat.0.suc.suc, a,
            matrix_smul[R, R](ring_as_module[R], Nat.0.suc.suc, Nat.0.suc.suc, lam, matrix_one[R](Nat.0.suc.suc))),
        v, fin_zero(Nat.0.suc))
    matrix_apply[R](Nat.0.suc.suc,
        matrix_sub[R](Nat.0.suc.suc, Nat.0.suc.suc, a,
            matrix_smul[R, R](ring_as_module[R], Nat.0.suc.suc, Nat.0.suc.suc, lam, matrix_one[R](Nat.0.suc.suc))),
        v, fin_zero(Nat.0.suc)) =
        fin_sum[R](Nat.0.suc.suc, matrix_apply_term[R](Nat.0.suc.suc,
            matrix_sub[R](Nat.0.suc.suc, Nat.0.suc.suc, a,
                matrix_smul[R, R](ring_as_module[R], Nat.0.suc.suc, Nat.0.suc.suc, lam, matrix_one[R](Nat.0.suc.suc))),
            v, fin_zero(Nat.0.suc)))
    fin_sum_two[R](matrix_apply_term[R](Nat.0.suc.suc,
        matrix_sub[R](Nat.0.suc.suc, Nat.0.suc.suc, a,
            matrix_smul[R, R](ring_as_module[R], Nat.0.suc.suc, Nat.0.suc.suc, lam, matrix_one[R](Nat.0.suc.suc))),
        v, fin_zero(Nat.0.suc)))
    matrix_apply[R](Nat.0.suc.suc,
        matrix_sub[R](Nat.0.suc.suc, Nat.0.suc.suc, a,
            matrix_smul[R, R](ring_as_module[R], Nat.0.suc.suc, Nat.0.suc.suc, lam, matrix_one[R](Nat.0.suc.suc))),
        v, fin_zero(Nat.0.suc)) =
        matrix_apply_term[R](Nat.0.suc.suc,
            matrix_sub[R](Nat.0.suc.suc, Nat.0.suc.suc, a,
                matrix_smul[R, R](ring_as_module[R], Nat.0.suc.suc, Nat.0.suc.suc, lam, matrix_one[R](Nat.0.suc.suc))),
            v, fin_zero(Nat.0.suc), fin_zero(Nat.0.suc)) +
        matrix_apply_term[R](Nat.0.suc.suc,
            matrix_sub[R](Nat.0.suc.suc, Nat.0.suc.suc, a,
                matrix_smul[R, R](ring_as_module[R], Nat.0.suc.suc, Nat.0.suc.suc, lam, matrix_one[R](Nat.0.suc.suc))),
            v, fin_zero(Nat.0.suc), fin_succ(Nat.0.suc, fin_zero(Nat.0)))
    matrix_apply_term[R](Nat.0.suc.suc,
        matrix_sub[R](Nat.0.suc.suc, Nat.0.suc.suc, a,
            matrix_smul[R, R](ring_as_module[R], Nat.0.suc.suc, Nat.0.suc.suc, lam, matrix_one[R](Nat.0.suc.suc))),
        v, fin_zero(Nat.0.suc), fin_zero(Nat.0.suc)) =
        matrix_entry[R](Nat.0.suc.suc, Nat.0.suc.suc,
            matrix_sub[R](Nat.0.suc.suc, Nat.0.suc.suc, a,
                matrix_smul[R, R](ring_as_module[R], Nat.0.suc.suc, Nat.0.suc.suc, lam, matrix_one[R](Nat.0.suc.suc))),
            fin_zero(Nat.0.suc), fin_zero(Nat.0.suc)) * v(fin_zero(Nat.0.suc))
    matrix_apply_term[R](Nat.0.suc.suc,
        matrix_sub[R](Nat.0.suc.suc, Nat.0.suc.suc, a,
            matrix_smul[R, R](ring_as_module[R], Nat.0.suc.suc, Nat.0.suc.suc, lam, matrix_one[R](Nat.0.suc.suc))),
        v, fin_zero(Nat.0.suc), fin_succ(Nat.0.suc, fin_zero(Nat.0))) =
        matrix_entry[R](Nat.0.suc.suc, Nat.0.suc.suc,
            matrix_sub[R](Nat.0.suc.suc, Nat.0.suc.suc, a,
                matrix_smul[R, R](ring_as_module[R], Nat.0.suc.suc, Nat.0.suc.suc, lam, matrix_one[R](Nat.0.suc.suc))),
            fin_zero(Nat.0.suc), fin_succ(Nat.0.suc, fin_zero(Nat.0))) * v(fin_succ(Nat.0.suc, fin_zero(Nat.0)))
    char_poly_matrix_entry_00[R](a, lam)
    char_poly_matrix_entry_01[R](a, lam)
    matrix_apply[R](Nat.0.suc.suc,
        matrix_sub[R](Nat.0.suc.suc, Nat.0.suc.suc, a,
            matrix_smul[R, R](ring_as_module[R], Nat.0.suc.suc, Nat.0.suc.suc, lam, matrix_one[R](Nat.0.suc.suc))),
        v, fin_zero(Nat.0.suc)) =
        (matrix2_00[R](Nat.0, a) - lam) * v(fin_zero(Nat.0.suc)) + matrix2_01[R](Nat.0, a) * v(fin_succ(Nat.0.suc, fin_zero(Nat.0)))
    matrix2_apply_entry[R](a, v, fin_zero(Nat.0.suc))
    matrix_apply[R](Nat.0.suc.suc, a, v, fin_zero(Nat.0.suc)) =
        matrix2_00[R](Nat.0, a) * v(fin_zero(Nat.0.suc)) + matrix2_01[R](Nat.0, a) * v(fin_succ(Nat.0.suc, fin_zero(Nat.0)))
    mul_sub_right[R](matrix2_00[R](Nat.0, a), lam, v(fin_zero(Nat.0.suc)))
    (matrix2_00[R](Nat.0, a) - lam) * v(fin_zero(Nat.0.suc)) =
        matrix2_00[R](Nat.0, a) * v(fin_zero(Nat.0.suc)) - lam * v(fin_zero(Nat.0.suc))
    ring_move_neg_to_back[R](matrix2_00[R](Nat.0, a) * v(fin_zero(Nat.0.suc)),
        matrix2_01[R](Nat.0, a) * v(fin_succ(Nat.0.suc, fin_zero(Nat.0))), lam * v(fin_zero(Nat.0.suc)))
    (matrix2_00[R](Nat.0, a) * v(fin_zero(Nat.0.suc)) - lam * v(fin_zero(Nat.0.suc))) +
        matrix2_01[R](Nat.0, a) * v(fin_succ(Nat.0.suc, fin_zero(Nat.0))) =
        matrix2_00[R](Nat.0, a) * v(fin_zero(Nat.0.suc)) + matrix2_01[R](Nat.0, a) * v(fin_succ(Nat.0.suc, fin_zero(Nat.0))) -
            lam * v(fin_zero(Nat.0.suc))
    (matrix2_00[R](Nat.0, a) - lam) * v(fin_zero(Nat.0.suc)) + matrix2_01[R](Nat.0, a) * v(fin_succ(Nat.0.suc, fin_zero(Nat.0))) =
        matrix2_00[R](Nat.0, a) * v(fin_zero(Nat.0.suc)) + matrix2_01[R](Nat.0, a) * v(fin_succ(Nat.0.suc, fin_zero(Nat.0))) -
            lam * v(fin_zero(Nat.0.suc))
    matrix_apply[R](Nat.0.suc.suc,
        matrix_sub[R](Nat.0.suc.suc, Nat.0.suc.suc, a,
            matrix_smul[R, R](ring_as_module[R], Nat.0.suc.suc, Nat.0.suc.suc, lam, matrix_one[R](Nat.0.suc.suc))),
        v, fin_zero(Nat.0.suc)) =
        matrix_apply[R](Nat.0.suc.suc, a, v, fin_zero(Nat.0.suc)) - lam * v(fin_zero(Nat.0.suc))
}

/// Applying `A - lam * I` to a vector at the second coordinate gives the
/// application of `A` minus `lam` times the coordinate.
theorem matrix_apply_char_poly_sub_1[R: CommRing](
    a: Matrix[R, Nat.0.suc.suc, Nat.0.suc.suc], lam: R, v: Fin[Nat.0.suc.suc] -> R
) {
    matrix_apply[R](Nat.0.suc.suc,
        matrix_sub[R](Nat.0.suc.suc, Nat.0.suc.suc, a,
            matrix_smul[R, R](ring_as_module[R], Nat.0.suc.suc, Nat.0.suc.suc, lam, matrix_one[R](Nat.0.suc.suc))),
        v, fin_succ(Nat.0.suc, fin_zero(Nat.0))) =
        matrix_apply[R](Nat.0.suc.suc, a, v, fin_succ(Nat.0.suc, fin_zero(Nat.0))) - lam * v(fin_succ(Nat.0.suc, fin_zero(Nat.0)))
} by {
    matrix_apply_eq[R](Nat.0.suc.suc,
        matrix_sub[R](Nat.0.suc.suc, Nat.0.suc.suc, a,
            matrix_smul[R, R](ring_as_module[R], Nat.0.suc.suc, Nat.0.suc.suc, lam, matrix_one[R](Nat.0.suc.suc))),
        v, fin_succ(Nat.0.suc, fin_zero(Nat.0)))
    matrix_apply[R](Nat.0.suc.suc,
        matrix_sub[R](Nat.0.suc.suc, Nat.0.suc.suc, a,
            matrix_smul[R, R](ring_as_module[R], Nat.0.suc.suc, Nat.0.suc.suc, lam, matrix_one[R](Nat.0.suc.suc))),
        v, fin_succ(Nat.0.suc, fin_zero(Nat.0))) =
        fin_sum[R](Nat.0.suc.suc, matrix_apply_term[R](Nat.0.suc.suc,
            matrix_sub[R](Nat.0.suc.suc, Nat.0.suc.suc, a,
                matrix_smul[R, R](ring_as_module[R], Nat.0.suc.suc, Nat.0.suc.suc, lam, matrix_one[R](Nat.0.suc.suc))),
            v, fin_succ(Nat.0.suc, fin_zero(Nat.0))))
    fin_sum_two[R](matrix_apply_term[R](Nat.0.suc.suc,
        matrix_sub[R](Nat.0.suc.suc, Nat.0.suc.suc, a,
            matrix_smul[R, R](ring_as_module[R], Nat.0.suc.suc, Nat.0.suc.suc, lam, matrix_one[R](Nat.0.suc.suc))),
        v, fin_succ(Nat.0.suc, fin_zero(Nat.0))))
    matrix_apply[R](Nat.0.suc.suc,
        matrix_sub[R](Nat.0.suc.suc, Nat.0.suc.suc, a,
            matrix_smul[R, R](ring_as_module[R], Nat.0.suc.suc, Nat.0.suc.suc, lam, matrix_one[R](Nat.0.suc.suc))),
        v, fin_succ(Nat.0.suc, fin_zero(Nat.0))) =
        matrix_apply_term[R](Nat.0.suc.suc,
            matrix_sub[R](Nat.0.suc.suc, Nat.0.suc.suc, a,
                matrix_smul[R, R](ring_as_module[R], Nat.0.suc.suc, Nat.0.suc.suc, lam, matrix_one[R](Nat.0.suc.suc))),
            v, fin_succ(Nat.0.suc, fin_zero(Nat.0)), fin_zero(Nat.0.suc)) +
        matrix_apply_term[R](Nat.0.suc.suc,
            matrix_sub[R](Nat.0.suc.suc, Nat.0.suc.suc, a,
                matrix_smul[R, R](ring_as_module[R], Nat.0.suc.suc, Nat.0.suc.suc, lam, matrix_one[R](Nat.0.suc.suc))),
            v, fin_succ(Nat.0.suc, fin_zero(Nat.0)), fin_succ(Nat.0.suc, fin_zero(Nat.0)))
    matrix_apply_term[R](Nat.0.suc.suc,
        matrix_sub[R](Nat.0.suc.suc, Nat.0.suc.suc, a,
            matrix_smul[R, R](ring_as_module[R], Nat.0.suc.suc, Nat.0.suc.suc, lam, matrix_one[R](Nat.0.suc.suc))),
        v, fin_succ(Nat.0.suc, fin_zero(Nat.0)), fin_zero(Nat.0.suc)) =
        matrix_entry[R](Nat.0.suc.suc, Nat.0.suc.suc,
            matrix_sub[R](Nat.0.suc.suc, Nat.0.suc.suc, a,
                matrix_smul[R, R](ring_as_module[R], Nat.0.suc.suc, Nat.0.suc.suc, lam, matrix_one[R](Nat.0.suc.suc))),
            fin_succ(Nat.0.suc, fin_zero(Nat.0)), fin_zero(Nat.0.suc)) * v(fin_zero(Nat.0.suc))
    matrix_apply_term[R](Nat.0.suc.suc,
        matrix_sub[R](Nat.0.suc.suc, Nat.0.suc.suc, a,
            matrix_smul[R, R](ring_as_module[R], Nat.0.suc.suc, Nat.0.suc.suc, lam, matrix_one[R](Nat.0.suc.suc))),
        v, fin_succ(Nat.0.suc, fin_zero(Nat.0)), fin_succ(Nat.0.suc, fin_zero(Nat.0))) =
        matrix_entry[R](Nat.0.suc.suc, Nat.0.suc.suc,
            matrix_sub[R](Nat.0.suc.suc, Nat.0.suc.suc, a,
                matrix_smul[R, R](ring_as_module[R], Nat.0.suc.suc, Nat.0.suc.suc, lam, matrix_one[R](Nat.0.suc.suc))),
            fin_succ(Nat.0.suc, fin_zero(Nat.0)), fin_succ(Nat.0.suc, fin_zero(Nat.0))) * v(fin_succ(Nat.0.suc, fin_zero(Nat.0)))
    char_poly_matrix_entry_10[R](a, lam)
    char_poly_matrix_entry_11[R](a, lam)
    matrix_apply[R](Nat.0.suc.suc,
        matrix_sub[R](Nat.0.suc.suc, Nat.0.suc.suc, a,
            matrix_smul[R, R](ring_as_module[R], Nat.0.suc.suc, Nat.0.suc.suc, lam, matrix_one[R](Nat.0.suc.suc))),
        v, fin_succ(Nat.0.suc, fin_zero(Nat.0))) =
        matrix2_10[R](Nat.0, a) * v(fin_zero(Nat.0.suc)) + (matrix2_11[R](Nat.0, a) - lam) * v(fin_succ(Nat.0.suc, fin_zero(Nat.0)))
    matrix2_apply_entry[R](a, v, fin_succ(Nat.0.suc, fin_zero(Nat.0)))
    matrix_apply[R](Nat.0.suc.suc, a, v, fin_succ(Nat.0.suc, fin_zero(Nat.0))) =
        matrix2_10[R](Nat.0, a) * v(fin_zero(Nat.0.suc)) + matrix2_11[R](Nat.0, a) * v(fin_succ(Nat.0.suc, fin_zero(Nat.0)))
    mul_sub_right[R](matrix2_11[R](Nat.0, a), lam, v(fin_succ(Nat.0.suc, fin_zero(Nat.0))))
    (matrix2_11[R](Nat.0, a) - lam) * v(fin_succ(Nat.0.suc, fin_zero(Nat.0))) =
        matrix2_11[R](Nat.0, a) * v(fin_succ(Nat.0.suc, fin_zero(Nat.0))) - lam * v(fin_succ(Nat.0.suc, fin_zero(Nat.0)))
    matrix2_10[R](Nat.0, a) * v(fin_zero(Nat.0.suc)) + (matrix2_11[R](Nat.0, a) - lam) * v(fin_succ(Nat.0.suc, fin_zero(Nat.0))) =
        matrix2_10[R](Nat.0, a) * v(fin_zero(Nat.0.suc)) + matrix2_11[R](Nat.0, a) * v(fin_succ(Nat.0.suc, fin_zero(Nat.0))) -
            lam * v(fin_succ(Nat.0.suc, fin_zero(Nat.0)))
    matrix_apply[R](Nat.0.suc.suc,
        matrix_sub[R](Nat.0.suc.suc, Nat.0.suc.suc, a,
            matrix_smul[R, R](ring_as_module[R], Nat.0.suc.suc, Nat.0.suc.suc, lam, matrix_one[R](Nat.0.suc.suc))),
        v, fin_succ(Nat.0.suc, fin_zero(Nat.0))) =
        matrix_apply[R](Nat.0.suc.suc, a, v, fin_succ(Nat.0.suc, fin_zero(Nat.0))) - lam * v(fin_succ(Nat.0.suc, fin_zero(Nat.0)))
}

/// Applying `A - lam * I` to a vector is applying `A` and subtracting `lam`
/// times the vector.
theorem matrix_apply_char_poly_sub[R: CommRing](
    a: Matrix[R, Nat.0.suc.suc, Nat.0.suc.suc], lam: R, v: Fin[Nat.0.suc.suc] -> R, i: Fin[Nat.0.suc.suc]
) {
    matrix_apply[R](Nat.0.suc.suc,
        matrix_sub[R](Nat.0.suc.suc, Nat.0.suc.suc, a,
            matrix_smul[R, R](ring_as_module[R], Nat.0.suc.suc, Nat.0.suc.suc, lam, matrix_one[R](Nat.0.suc.suc))),
        v, i) =
        matrix_apply[R](Nat.0.suc.suc, a, v, i) - lam * v(i)
} by {
    fin_two_index_cases(i)
    if i.value = Nat.0 {
        fin_two_zero_value_two
        i.value = fin_zero(Nat.0.suc).value
        ext(Nat.0.suc.suc, i, fin_zero(Nat.0.suc))
        i = fin_zero(Nat.0.suc)
        matrix_apply_char_poly_sub_0[R](a, lam, v)
        matrix_apply[R](Nat.0.suc.suc,
            matrix_sub[R](Nat.0.suc.suc, Nat.0.suc.suc, a,
                matrix_smul[R, R](ring_as_module[R], Nat.0.suc.suc, Nat.0.suc.suc, lam, matrix_one[R](Nat.0.suc.suc))),
            v, fin_zero(Nat.0.suc)) =
            matrix_apply[R](Nat.0.suc.suc, a, v, fin_zero(Nat.0.suc)) - lam * v(fin_zero(Nat.0.suc))
        matrix_apply[R](Nat.0.suc.suc,
            matrix_sub[R](Nat.0.suc.suc, Nat.0.suc.suc, a,
                matrix_smul[R, R](ring_as_module[R], Nat.0.suc.suc, Nat.0.suc.suc, lam, matrix_one[R](Nat.0.suc.suc))),
            v, i) =
            matrix_apply[R](Nat.0.suc.suc, a, v, i) - lam * v(i)
    } else {
        fin_two_one_value_two
        i.value = fin_succ(Nat.0.suc, fin_zero(Nat.0)).value
        ext(Nat.0.suc.suc, i, fin_succ(Nat.0.suc, fin_zero(Nat.0)))
        i = fin_succ(Nat.0.suc, fin_zero(Nat.0))
        matrix_apply_char_poly_sub_1[R](a, lam, v)
        matrix_apply[R](Nat.0.suc.suc,
            matrix_sub[R](Nat.0.suc.suc, Nat.0.suc.suc, a,
                matrix_smul[R, R](ring_as_module[R], Nat.0.suc.suc, Nat.0.suc.suc, lam, matrix_one[R](Nat.0.suc.suc))),
            v, fin_succ(Nat.0.suc, fin_zero(Nat.0))) =
            matrix_apply[R](Nat.0.suc.suc, a, v, fin_succ(Nat.0.suc, fin_zero(Nat.0))) - lam * v(fin_succ(Nat.0.suc, fin_zero(Nat.0)))
        matrix_apply[R](Nat.0.suc.suc,
            matrix_sub[R](Nat.0.suc.suc, Nat.0.suc.suc, a,
                matrix_smul[R, R](ring_as_module[R], Nat.0.suc.suc, Nat.0.suc.suc, lam, matrix_one[R](Nat.0.suc.suc))),
            v, i) =
            matrix_apply[R](Nat.0.suc.suc, a, v, i) - lam * v(i)
    }
}

/// The characteristic polynomial is the two-by-two determinant of `A - lam * I`.
theorem char_poly_det_eq_matrix2_det[R: Ring](a: Matrix[R, Nat.0.suc.suc, Nat.0.suc.suc], lam: R) {
    char_poly_det[R](Nat.0, a, lam) = matrix2_det[R](Nat.0,
        matrix_sub[R](Nat.0.suc.suc, Nat.0.suc.suc, a,
            matrix_smul[R, R](ring_as_module[R], Nat.0.suc.suc, Nat.0.suc.suc, lam, matrix_one[R](Nat.0.suc.suc))))
} by {
    char_poly_det_eq_entries[R](a, lam)
    char_poly_det[R](Nat.0, a, lam) = char_poly_2x2[R](Nat.0, a, lam)
    char_poly_2x2[R](Nat.0, a, lam) = (matrix2_00[R](Nat.0, a) - lam) * (matrix2_11[R](Nat.0, a) - lam) -
        matrix2_01[R](Nat.0, a) * matrix2_10[R](Nat.0, a)
    matrix2_det_unfold[R](matrix_sub[R](Nat.0.suc.suc, Nat.0.suc.suc, a,
        matrix_smul[R, R](ring_as_module[R], Nat.0.suc.suc, Nat.0.suc.suc, lam, matrix_one[R](Nat.0.suc.suc))))
    matrix2_det[R](Nat.0, matrix_sub[R](Nat.0.suc.suc, Nat.0.suc.suc, a,
        matrix_smul[R, R](ring_as_module[R], Nat.0.suc.suc, Nat.0.suc.suc, lam, matrix_one[R](Nat.0.suc.suc)))) =
        matrix2_entry_00[R](Nat.0, matrix_sub[R](Nat.0.suc.suc, Nat.0.suc.suc, a,
            matrix_smul[R, R](ring_as_module[R], Nat.0.suc.suc, Nat.0.suc.suc, lam, matrix_one[R](Nat.0.suc.suc)))) *
            matrix2_entry_11[R](Nat.0, matrix_sub[R](Nat.0.suc.suc, Nat.0.suc.suc, a,
                matrix_smul[R, R](ring_as_module[R], Nat.0.suc.suc, Nat.0.suc.suc, lam, matrix_one[R](Nat.0.suc.suc)))) -
        matrix2_entry_01[R](Nat.0, matrix_sub[R](Nat.0.suc.suc, Nat.0.suc.suc, a,
            matrix_smul[R, R](ring_as_module[R], Nat.0.suc.suc, Nat.0.suc.suc, lam, matrix_one[R](Nat.0.suc.suc)))) *
            matrix2_entry_10[R](Nat.0, matrix_sub[R](Nat.0.suc.suc, Nat.0.suc.suc, a,
                matrix_smul[R, R](ring_as_module[R], Nat.0.suc.suc, Nat.0.suc.suc, lam, matrix_one[R](Nat.0.suc.suc))))
    char_poly_matrix_entry_00[R](a, lam)
    char_poly_matrix_entry_01[R](a, lam)
    char_poly_matrix_entry_10[R](a, lam)
    char_poly_matrix_entry_11[R](a, lam)
    char_poly_det[R](Nat.0, a, lam) = matrix2_det[R](Nat.0,
        matrix_sub[R](Nat.0.suc.suc, Nat.0.suc.suc, a,
            matrix_smul[R, R](ring_as_module[R], Nat.0.suc.suc, Nat.0.suc.suc, lam, matrix_one[R](Nat.0.suc.suc))))
}

/// An eigenvalue of a two-by-two matrix annihilates the characteristic
/// polynomial: `lam` is an eigenvalue of `A` implies `det(A - lam * I) = 0`.
theorem eigenvalue_implies_char_poly_zero[F: Field](a: Matrix[F, Nat.0.suc.suc, Nat.0.suc.suc], lam: F) {
    is_eigenvalue[F](Nat.0.suc.suc, a, lam) implies char_poly_det[F](Nat.0, a, lam) = F.0
} by {
    if is_eigenvalue[F](Nat.0.suc.suc, a, lam) {
        is_eigenvalue_witness[F](Nat.0.suc.suc, a, lam)
        let (v: Fin[Nat.0.suc.suc] -> F) satisfy {
            is_eigenvector[F](Nat.0.suc.suc, a, lam, v)
        }
        is_eigenvector_nonzero[F](Nat.0.suc.suc, a, lam, v)
        is_nonzero_vector[F](Nat.0.suc.suc, v)
        is_nonzero_vector_witness[F](Nat.0.suc.suc, v)
        let (j: Fin[Nat.0.suc.suc]) satisfy { v(j) != F.0 }
        forall(i: Fin[Nat.0.suc.suc]) {
            matrix_apply_char_poly_sub[F](a, lam, v, i)
            matrix_apply[F](Nat.0.suc.suc,
                matrix_sub[F](Nat.0.suc.suc, Nat.0.suc.suc, a,
                    matrix_smul[F, F](ring_as_module[F], Nat.0.suc.suc, Nat.0.suc.suc, lam, matrix_one[F](Nat.0.suc.suc))),
                v, i) =
                matrix_apply[F](Nat.0.suc.suc, a, v, i) - lam * v(i)
            is_eigenvector_apply[F](Nat.0.suc.suc, a, lam, v, i)
            matrix_apply[F](Nat.0.suc.suc, a, v, i) = lam * v(i)
            matrix_apply[F](Nat.0.suc.suc,
                matrix_sub[F](Nat.0.suc.suc, Nat.0.suc.suc, a,
                    matrix_smul[F, F](ring_as_module[F], Nat.0.suc.suc, Nat.0.suc.suc, lam, matrix_one[F](Nat.0.suc.suc))),
                v, i) = lam * v(i) - lam * v(i)
            matrix_apply[F](Nat.0.suc.suc,
                matrix_sub[F](Nat.0.suc.suc, Nat.0.suc.suc, a,
                    matrix_smul[F, F](ring_as_module[F], Nat.0.suc.suc, Nat.0.suc.suc, lam, matrix_one[F](Nat.0.suc.suc))),
                v, i) = F.0
        }
        if char_poly_det[F](Nat.0, a, lam) = F.0 {
            char_poly_det[F](Nat.0, a, lam) = F.0
        } else {
            char_poly_det[F](Nat.0, a, lam) != F.0
            char_poly_det_eq_matrix2_det[F](a, lam)
            char_poly_det[F](Nat.0, a, lam) = matrix2_det[F](Nat.0,
                matrix_sub[F](Nat.0.suc.suc, Nat.0.suc.suc, a,
                    matrix_smul[F, F](ring_as_module[F], Nat.0.suc.suc, Nat.0.suc.suc, lam, matrix_one[F](Nat.0.suc.suc))))
            matrix2_det[F](Nat.0,
                matrix_sub[F](Nat.0.suc.suc, Nat.0.suc.suc, a,
                    matrix_smul[F, F](ring_as_module[F], Nat.0.suc.suc, Nat.0.suc.suc, lam, matrix_one[F](Nat.0.suc.suc)))) != F.0
            matrix2_det_nonzero_implies_invertible[F](
                matrix_sub[F](Nat.0.suc.suc, Nat.0.suc.suc, a,
                    matrix_smul[F, F](ring_as_module[F], Nat.0.suc.suc, Nat.0.suc.suc, lam, matrix_one[F](Nat.0.suc.suc))))
            exists(b: Matrix[F, Nat.0.suc.suc, Nat.0.suc.suc]) {
                square_matrix_mul[F](Nat.0.suc.suc,
                    matrix_sub[F](Nat.0.suc.suc, Nat.0.suc.suc, a,
                        matrix_smul[F, F](ring_as_module[F], Nat.0.suc.suc, Nat.0.suc.suc, lam, matrix_one[F](Nat.0.suc.suc))),
                    b) = matrix_one[F](Nat.0.suc.suc) and
                square_matrix_mul[F](Nat.0.suc.suc, b,
                    matrix_sub[F](Nat.0.suc.suc, Nat.0.suc.suc, a,
                        matrix_smul[F, F](ring_as_module[F], Nat.0.suc.suc, Nat.0.suc.suc, lam, matrix_one[F](Nat.0.suc.suc)))) = matrix_one[F](Nat.0.suc.suc)
            }
            let (b: Matrix[F, Nat.0.suc.suc, Nat.0.suc.suc]) satisfy {
                square_matrix_mul[F](Nat.0.suc.suc,
                    matrix_sub[F](Nat.0.suc.suc, Nat.0.suc.suc, a,
                        matrix_smul[F, F](ring_as_module[F], Nat.0.suc.suc, Nat.0.suc.suc, lam, matrix_one[F](Nat.0.suc.suc))),
                    b) = matrix_one[F](Nat.0.suc.suc) and
                square_matrix_mul[F](Nat.0.suc.suc, b,
                    matrix_sub[F](Nat.0.suc.suc, Nat.0.suc.suc, a,
                        matrix_smul[F, F](ring_as_module[F], Nat.0.suc.suc, Nat.0.suc.suc, lam, matrix_one[F](Nat.0.suc.suc)))) = matrix_one[F](Nat.0.suc.suc)
            }
            matrix_apply_mul_two[F](b,
                matrix_sub[F](Nat.0.suc.suc, Nat.0.suc.suc, a,
                    matrix_smul[F, F](ring_as_module[F], Nat.0.suc.suc, Nat.0.suc.suc, lam, matrix_one[F](Nat.0.suc.suc))),
                v, j)
            matrix_apply[F](Nat.0.suc.suc, square_matrix_mul[F](Nat.0.suc.suc, b,
                matrix_sub[F](Nat.0.suc.suc, Nat.0.suc.suc, a,
                    matrix_smul[F, F](ring_as_module[F], Nat.0.suc.suc, Nat.0.suc.suc, lam, matrix_one[F](Nat.0.suc.suc)))),
                v, j) =
                matrix_apply[F](Nat.0.suc.suc, b, matrix_apply_vec[F](Nat.0.suc.suc,
                    matrix_sub[F](Nat.0.suc.suc, Nat.0.suc.suc, a,
                        matrix_smul[F, F](ring_as_module[F], Nat.0.suc.suc, Nat.0.suc.suc, lam, matrix_one[F](Nat.0.suc.suc))),
                    v), j)
            matrix_apply[F](Nat.0.suc.suc, matrix_one[F](Nat.0.suc.suc), v, j) = v(j)
            matrix_apply[F](Nat.0.suc.suc, square_matrix_mul[F](Nat.0.suc.suc, b,
                matrix_sub[F](Nat.0.suc.suc, Nat.0.suc.suc, a,
                    matrix_smul[F, F](ring_as_module[F], Nat.0.suc.suc, Nat.0.suc.suc, lam, matrix_one[F](Nat.0.suc.suc)))),
                v, j) = matrix_apply[F](Nat.0.suc.suc, matrix_one[F](Nat.0.suc.suc), v, j)
            matrix_apply[F](Nat.0.suc.suc, b, matrix_apply_vec[F](Nat.0.suc.suc,
                matrix_sub[F](Nat.0.suc.suc, Nat.0.suc.suc, a,
                    matrix_smul[F, F](ring_as_module[F], Nat.0.suc.suc, Nat.0.suc.suc, lam, matrix_one[F](Nat.0.suc.suc))),
                v), j) = v(j)
            forall(k: Fin[Nat.0.suc.suc]) {
                matrix_apply[F](Nat.0.suc.suc,
                    matrix_sub[F](Nat.0.suc.suc, Nat.0.suc.suc, a,
                        matrix_smul[F, F](ring_as_module[F], Nat.0.suc.suc, Nat.0.suc.suc, lam, matrix_one[F](Nat.0.suc.suc))),
                    v, k) = F.0
                matrix_apply_vec[F](Nat.0.suc.suc,
                    matrix_sub[F](Nat.0.suc.suc, Nat.0.suc.suc, a,
                        matrix_smul[F, F](ring_as_module[F], Nat.0.suc.suc, Nat.0.suc.suc, lam, matrix_one[F](Nat.0.suc.suc))),
                    v, k) =
                    matrix_apply[F](Nat.0.suc.suc,
                        matrix_sub[F](Nat.0.suc.suc, Nat.0.suc.suc, a,
                            matrix_smul[F, F](ring_as_module[F], Nat.0.suc.suc, Nat.0.suc.suc, lam, matrix_one[F](Nat.0.suc.suc))),
                        v, k)
                fin_vector_zero_apply[F](Nat.0.suc.suc, k)
                fin_vector_zero[F](Nat.0.suc.suc, k) = F.0
                matrix_apply_vec[F](Nat.0.suc.suc,
                    matrix_sub[F](Nat.0.suc.suc, Nat.0.suc.suc, a,
                        matrix_smul[F, F](ring_as_module[F], Nat.0.suc.suc, Nat.0.suc.suc, lam, matrix_one[F](Nat.0.suc.suc))),
                    v, k) = fin_vector_zero[F](Nat.0.suc.suc, k)
            }
            matrix_apply_pointwise_eq[F](Nat.0.suc.suc, b, matrix_apply_vec[F](Nat.0.suc.suc,
                matrix_sub[F](Nat.0.suc.suc, Nat.0.suc.suc, a,
                    matrix_smul[F, F](ring_as_module[F], Nat.0.suc.suc, Nat.0.suc.suc, lam, matrix_one[F](Nat.0.suc.suc))),
                v), fin_vector_zero[F](Nat.0.suc.suc), j)
            matrix_apply[F](Nat.0.suc.suc, b, matrix_apply_vec[F](Nat.0.suc.suc,
                matrix_sub[F](Nat.0.suc.suc, Nat.0.suc.suc, a,
                    matrix_smul[F, F](ring_as_module[F], Nat.0.suc.suc, Nat.0.suc.suc, lam, matrix_one[F](Nat.0.suc.suc))),
                v), j) =
                matrix_apply[F](Nat.0.suc.suc, b, fin_vector_zero[F](Nat.0.suc.suc), j)
            matrix_apply_zero[F](Nat.0.suc.suc, b, j)
            matrix_apply[F](Nat.0.suc.suc, b, fin_vector_zero[F](Nat.0.suc.suc), j) = F.0
            matrix_apply[F](Nat.0.suc.suc, b, matrix_apply_vec[F](Nat.0.suc.suc,
                matrix_sub[F](Nat.0.suc.suc, Nat.0.suc.suc, a,
                    matrix_smul[F, F](ring_as_module[F], Nat.0.suc.suc, Nat.0.suc.suc, lam, matrix_one[F](Nat.0.suc.suc))),
                v), j) = F.0
            v(j) = F.0
            false
        }
        char_poly_det[F](Nat.0, a, lam) = F.0
    }
}

/// The coordinate vector `(x0, x1)` evaluates to `x0` at the first index.
theorem matrix2_vec_entry_0[R](x0: R, x1: R) {
    matrix2_vec[R](Nat.0.suc.suc, x0, x1, fin_zero(Nat.0.suc)) = x0
} by {
    matrix2_vec[R](Nat.0.suc.suc, x0, x1, fin_zero(Nat.0.suc)) =
        if fin_zero(Nat.0.suc).value = Nat.0 { x0 } else { x1 }
    fin_zero_value(Nat.0.suc)
    fin_zero(Nat.0.suc).value = Nat.0
    matrix2_vec[R](Nat.0.suc.suc, x0, x1, fin_zero(Nat.0.suc)) = x0
}

/// The coordinate vector `(x0, x1)` evaluates to `x1` at the second index.
theorem matrix2_vec_entry_1[R](x0: R, x1: R) {
    matrix2_vec[R](Nat.0.suc.suc, x0, x1, fin_succ(Nat.0.suc, fin_zero(Nat.0))) = x1
} by {
    matrix2_vec[R](Nat.0.suc.suc, x0, x1, fin_succ(Nat.0.suc, fin_zero(Nat.0))) =
        if fin_succ(Nat.0.suc, fin_zero(Nat.0)).value = Nat.0 { x0 } else { x1 }
    fin_succ_value(Nat.0.suc, fin_zero(Nat.0))
    fin_succ(Nat.0.suc, fin_zero(Nat.0)).value = Nat.0.suc
    Nat.0 != Nat.0.suc
    if fin_succ(Nat.0.suc, fin_zero(Nat.0)).value = Nat.0 {
        Nat.0.suc = Nat.0
        false
    }
    matrix2_vec[R](Nat.0.suc.suc, x0, x1, fin_succ(Nat.0.suc, fin_zero(Nat.0))) = x1
}

/// The first row of `(A - lam * I)` applied to the coordinate vector
/// `(x, y)` is `(a00 - lam) * x + a01 * y`.
theorem matrix_apply_B_vec_0[R: Ring](
    a: Matrix[R, Nat.0.suc.suc, Nat.0.suc.suc], lam: R, x: R, y: R
) {
    matrix_apply[R](Nat.0.suc.suc,
        matrix_sub[R](Nat.0.suc.suc, Nat.0.suc.suc, a,
            matrix_smul[R, R](ring_as_module[R], Nat.0.suc.suc, Nat.0.suc.suc, lam, matrix_one[R](Nat.0.suc.suc))),
        matrix2_vec[R](Nat.0.suc.suc, x, y), fin_zero(Nat.0.suc)) =
        (matrix2_00[R](Nat.0, a) - lam) * x + matrix2_01[R](Nat.0, a) * y
} by {
    matrix_apply_eq[R](Nat.0.suc.suc,
        matrix_sub[R](Nat.0.suc.suc, Nat.0.suc.suc, a,
            matrix_smul[R, R](ring_as_module[R], Nat.0.suc.suc, Nat.0.suc.suc, lam, matrix_one[R](Nat.0.suc.suc))),
        matrix2_vec[R](Nat.0.suc.suc, x, y), fin_zero(Nat.0.suc))
    matrix_apply[R](Nat.0.suc.suc,
        matrix_sub[R](Nat.0.suc.suc, Nat.0.suc.suc, a,
            matrix_smul[R, R](ring_as_module[R], Nat.0.suc.suc, Nat.0.suc.suc, lam, matrix_one[R](Nat.0.suc.suc))),
        matrix2_vec[R](Nat.0.suc.suc, x, y), fin_zero(Nat.0.suc)) =
        fin_sum[R](Nat.0.suc.suc, matrix_apply_term[R](Nat.0.suc.suc,
            matrix_sub[R](Nat.0.suc.suc, Nat.0.suc.suc, a,
                matrix_smul[R, R](ring_as_module[R], Nat.0.suc.suc, Nat.0.suc.suc, lam, matrix_one[R](Nat.0.suc.suc))),
            matrix2_vec[R](Nat.0.suc.suc, x, y), fin_zero(Nat.0.suc)))
    fin_sum_two[R](matrix_apply_term[R](Nat.0.suc.suc,
        matrix_sub[R](Nat.0.suc.suc, Nat.0.suc.suc, a,
            matrix_smul[R, R](ring_as_module[R], Nat.0.suc.suc, Nat.0.suc.suc, lam, matrix_one[R](Nat.0.suc.suc))),
        matrix2_vec[R](Nat.0.suc.suc, x, y), fin_zero(Nat.0.suc)))
    matrix_apply[R](Nat.0.suc.suc,
        matrix_sub[R](Nat.0.suc.suc, Nat.0.suc.suc, a,
            matrix_smul[R, R](ring_as_module[R], Nat.0.suc.suc, Nat.0.suc.suc, lam, matrix_one[R](Nat.0.suc.suc))),
        matrix2_vec[R](Nat.0.suc.suc, x, y), fin_zero(Nat.0.suc)) =
        matrix_apply_term[R](Nat.0.suc.suc,
            matrix_sub[R](Nat.0.suc.suc, Nat.0.suc.suc, a,
                matrix_smul[R, R](ring_as_module[R], Nat.0.suc.suc, Nat.0.suc.suc, lam, matrix_one[R](Nat.0.suc.suc))),
            matrix2_vec[R](Nat.0.suc.suc, x, y), fin_zero(Nat.0.suc), fin_zero(Nat.0.suc)) +
        matrix_apply_term[R](Nat.0.suc.suc,
            matrix_sub[R](Nat.0.suc.suc, Nat.0.suc.suc, a,
                matrix_smul[R, R](ring_as_module[R], Nat.0.suc.suc, Nat.0.suc.suc, lam, matrix_one[R](Nat.0.suc.suc))),
            matrix2_vec[R](Nat.0.suc.suc, x, y), fin_zero(Nat.0.suc), fin_succ(Nat.0.suc, fin_zero(Nat.0)))
    matrix_apply_term[R](Nat.0.suc.suc,
        matrix_sub[R](Nat.0.suc.suc, Nat.0.suc.suc, a,
            matrix_smul[R, R](ring_as_module[R], Nat.0.suc.suc, Nat.0.suc.suc, lam, matrix_one[R](Nat.0.suc.suc))),
        matrix2_vec[R](Nat.0.suc.suc, x, y), fin_zero(Nat.0.suc), fin_zero(Nat.0.suc)) =
        matrix_entry[R](Nat.0.suc.suc, Nat.0.suc.suc,
            matrix_sub[R](Nat.0.suc.suc, Nat.0.suc.suc, a,
                matrix_smul[R, R](ring_as_module[R], Nat.0.suc.suc, Nat.0.suc.suc, lam, matrix_one[R](Nat.0.suc.suc))),
            fin_zero(Nat.0.suc), fin_zero(Nat.0.suc)) * matrix2_vec[R](Nat.0.suc.suc, x, y, fin_zero(Nat.0.suc))
    matrix_apply_term[R](Nat.0.suc.suc,
        matrix_sub[R](Nat.0.suc.suc, Nat.0.suc.suc, a,
            matrix_smul[R, R](ring_as_module[R], Nat.0.suc.suc, Nat.0.suc.suc, lam, matrix_one[R](Nat.0.suc.suc))),
        matrix2_vec[R](Nat.0.suc.suc, x, y), fin_zero(Nat.0.suc), fin_succ(Nat.0.suc, fin_zero(Nat.0))) =
        matrix_entry[R](Nat.0.suc.suc, Nat.0.suc.suc,
            matrix_sub[R](Nat.0.suc.suc, Nat.0.suc.suc, a,
                matrix_smul[R, R](ring_as_module[R], Nat.0.suc.suc, Nat.0.suc.suc, lam, matrix_one[R](Nat.0.suc.suc))),
            fin_zero(Nat.0.suc), fin_succ(Nat.0.suc, fin_zero(Nat.0))) * matrix2_vec[R](Nat.0.suc.suc, x, y, fin_succ(Nat.0.suc, fin_zero(Nat.0)))
    char_poly_matrix_entry_00[R](a, lam)
    char_poly_matrix_entry_01[R](a, lam)
    matrix2_vec_entry_0[R](x, y)
    matrix2_vec_entry_1[R](x, y)
    matrix_apply[R](Nat.0.suc.suc,
        matrix_sub[R](Nat.0.suc.suc, Nat.0.suc.suc, a,
            matrix_smul[R, R](ring_as_module[R], Nat.0.suc.suc, Nat.0.suc.suc, lam, matrix_one[R](Nat.0.suc.suc))),
        matrix2_vec[R](Nat.0.suc.suc, x, y), fin_zero(Nat.0.suc)) =
        (matrix2_00[R](Nat.0, a) - lam) * x + matrix2_01[R](Nat.0, a) * y
}

/// The second row of `(A - lam * I)` applied to the coordinate vector
/// `(x, y)` is `a10 * x + (a11 - lam) * y`.
theorem matrix_apply_B_vec_1[R: Ring](
    a: Matrix[R, Nat.0.suc.suc, Nat.0.suc.suc], lam: R, x: R, y: R
) {
    matrix_apply[R](Nat.0.suc.suc,
        matrix_sub[R](Nat.0.suc.suc, Nat.0.suc.suc, a,
            matrix_smul[R, R](ring_as_module[R], Nat.0.suc.suc, Nat.0.suc.suc, lam, matrix_one[R](Nat.0.suc.suc))),
        matrix2_vec[R](Nat.0.suc.suc, x, y), fin_succ(Nat.0.suc, fin_zero(Nat.0))) =
        matrix2_10[R](Nat.0, a) * x + (matrix2_11[R](Nat.0, a) - lam) * y
} by {
    matrix_apply_eq[R](Nat.0.suc.suc,
        matrix_sub[R](Nat.0.suc.suc, Nat.0.suc.suc, a,
            matrix_smul[R, R](ring_as_module[R], Nat.0.suc.suc, Nat.0.suc.suc, lam, matrix_one[R](Nat.0.suc.suc))),
        matrix2_vec[R](Nat.0.suc.suc, x, y), fin_succ(Nat.0.suc, fin_zero(Nat.0)))
    matrix_apply[R](Nat.0.suc.suc,
        matrix_sub[R](Nat.0.suc.suc, Nat.0.suc.suc, a,
            matrix_smul[R, R](ring_as_module[R], Nat.0.suc.suc, Nat.0.suc.suc, lam, matrix_one[R](Nat.0.suc.suc))),
        matrix2_vec[R](Nat.0.suc.suc, x, y), fin_succ(Nat.0.suc, fin_zero(Nat.0))) =
        fin_sum[R](Nat.0.suc.suc, matrix_apply_term[R](Nat.0.suc.suc,
            matrix_sub[R](Nat.0.suc.suc, Nat.0.suc.suc, a,
                matrix_smul[R, R](ring_as_module[R], Nat.0.suc.suc, Nat.0.suc.suc, lam, matrix_one[R](Nat.0.suc.suc))),
            matrix2_vec[R](Nat.0.suc.suc, x, y), fin_succ(Nat.0.suc, fin_zero(Nat.0))))
    fin_sum_two[R](matrix_apply_term[R](Nat.0.suc.suc,
        matrix_sub[R](Nat.0.suc.suc, Nat.0.suc.suc, a,
            matrix_smul[R, R](ring_as_module[R], Nat.0.suc.suc, Nat.0.suc.suc, lam, matrix_one[R](Nat.0.suc.suc))),
        matrix2_vec[R](Nat.0.suc.suc, x, y), fin_succ(Nat.0.suc, fin_zero(Nat.0))))
    matrix_apply[R](Nat.0.suc.suc,
        matrix_sub[R](Nat.0.suc.suc, Nat.0.suc.suc, a,
            matrix_smul[R, R](ring_as_module[R], Nat.0.suc.suc, Nat.0.suc.suc, lam, matrix_one[R](Nat.0.suc.suc))),
        matrix2_vec[R](Nat.0.suc.suc, x, y), fin_succ(Nat.0.suc, fin_zero(Nat.0))) =
        matrix_apply_term[R](Nat.0.suc.suc,
            matrix_sub[R](Nat.0.suc.suc, Nat.0.suc.suc, a,
                matrix_smul[R, R](ring_as_module[R], Nat.0.suc.suc, Nat.0.suc.suc, lam, matrix_one[R](Nat.0.suc.suc))),
            matrix2_vec[R](Nat.0.suc.suc, x, y), fin_succ(Nat.0.suc, fin_zero(Nat.0)), fin_zero(Nat.0.suc)) +
        matrix_apply_term[R](Nat.0.suc.suc,
            matrix_sub[R](Nat.0.suc.suc, Nat.0.suc.suc, a,
                matrix_smul[R, R](ring_as_module[R], Nat.0.suc.suc, Nat.0.suc.suc, lam, matrix_one[R](Nat.0.suc.suc))),
            matrix2_vec[R](Nat.0.suc.suc, x, y), fin_succ(Nat.0.suc, fin_zero(Nat.0)), fin_succ(Nat.0.suc, fin_zero(Nat.0)))
    matrix_apply_term[R](Nat.0.suc.suc,
        matrix_sub[R](Nat.0.suc.suc, Nat.0.suc.suc, a,
            matrix_smul[R, R](ring_as_module[R], Nat.0.suc.suc, Nat.0.suc.suc, lam, matrix_one[R](Nat.0.suc.suc))),
        matrix2_vec[R](Nat.0.suc.suc, x, y), fin_succ(Nat.0.suc, fin_zero(Nat.0)), fin_zero(Nat.0.suc)) =
        matrix_entry[R](Nat.0.suc.suc, Nat.0.suc.suc,
            matrix_sub[R](Nat.0.suc.suc, Nat.0.suc.suc, a,
                matrix_smul[R, R](ring_as_module[R], Nat.0.suc.suc, Nat.0.suc.suc, lam, matrix_one[R](Nat.0.suc.suc))),
            fin_succ(Nat.0.suc, fin_zero(Nat.0)), fin_zero(Nat.0.suc)) * matrix2_vec[R](Nat.0.suc.suc, x, y, fin_zero(Nat.0.suc))
    matrix_apply_term[R](Nat.0.suc.suc,
        matrix_sub[R](Nat.0.suc.suc, Nat.0.suc.suc, a,
            matrix_smul[R, R](ring_as_module[R], Nat.0.suc.suc, Nat.0.suc.suc, lam, matrix_one[R](Nat.0.suc.suc))),
        matrix2_vec[R](Nat.0.suc.suc, x, y), fin_succ(Nat.0.suc, fin_zero(Nat.0)), fin_succ(Nat.0.suc, fin_zero(Nat.0))) =
        matrix_entry[R](Nat.0.suc.suc, Nat.0.suc.suc,
            matrix_sub[R](Nat.0.suc.suc, Nat.0.suc.suc, a,
                matrix_smul[R, R](ring_as_module[R], Nat.0.suc.suc, Nat.0.suc.suc, lam, matrix_one[R](Nat.0.suc.suc))),
            fin_succ(Nat.0.suc, fin_zero(Nat.0)), fin_succ(Nat.0.suc, fin_zero(Nat.0))) * matrix2_vec[R](Nat.0.suc.suc, x, y, fin_succ(Nat.0.suc, fin_zero(Nat.0)))
    char_poly_matrix_entry_10[R](a, lam)
    char_poly_matrix_entry_11[R](a, lam)
    matrix2_vec_entry_0[R](x, y)
    matrix2_vec_entry_1[R](x, y)
    matrix_apply[R](Nat.0.suc.suc,
        matrix_sub[R](Nat.0.suc.suc, Nat.0.suc.suc, a,
            matrix_smul[R, R](ring_as_module[R], Nat.0.suc.suc, Nat.0.suc.suc, lam, matrix_one[R](Nat.0.suc.suc))),
        matrix2_vec[R](Nat.0.suc.suc, x, y), fin_succ(Nat.0.suc, fin_zero(Nat.0))) =
        matrix2_10[R](Nat.0, a) * x + (matrix2_11[R](Nat.0, a) - lam) * y
}

/// A nonzero first coordinate makes the coordinate vector nonzero.
theorem matrix2_vec_nonzero_0[F: Semiring](x: F, y: F) {
    x != F.0 implies is_nonzero_vector[F](Nat.0.suc.suc, matrix2_vec[F](Nat.0.suc.suc, x, y))
} by {
    if x != F.0 {
        matrix2_vec_entry_0[F](x, y)
        matrix2_vec[F](Nat.0.suc.suc, x, y, fin_zero(Nat.0.suc)) = x
        matrix2_vec[F](Nat.0.suc.suc, x, y)(fin_zero(Nat.0.suc)) != F.0
        is_nonzero_vector_intro[F](Nat.0.suc.suc, matrix2_vec[F](Nat.0.suc.suc, x, y), fin_zero(Nat.0.suc))
        is_nonzero_vector[F](Nat.0.suc.suc, matrix2_vec[F](Nat.0.suc.suc, x, y))
    }
}

/// A nonzero second coordinate makes the coordinate vector nonzero.
theorem matrix2_vec_nonzero_1[F: Semiring](x: F, y: F) {
    y != F.0 implies is_nonzero_vector[F](Nat.0.suc.suc, matrix2_vec[F](Nat.0.suc.suc, x, y))
} by {
    if y != F.0 {
        matrix2_vec_entry_1[F](x, y)
        matrix2_vec[F](Nat.0.suc.suc, x, y, fin_succ(Nat.0.suc, fin_zero(Nat.0))) = y
        matrix2_vec[F](Nat.0.suc.suc, x, y)(fin_succ(Nat.0.suc, fin_zero(Nat.0))) != F.0
        is_nonzero_vector_intro[F](Nat.0.suc.suc, matrix2_vec[F](Nat.0.suc.suc, x, y), fin_succ(Nat.0.suc, fin_zero(Nat.0)))
        is_nonzero_vector[F](Nat.0.suc.suc, matrix2_vec[F](Nat.0.suc.suc, x, y))
    }
}

/// The characteristic polynomial of a two-by-two matrix vanishes exactly when
/// its entrywise determinant form does.
theorem char_poly_det_entry_zero[R: Ring](a: Matrix[R, Nat.0.suc.suc, Nat.0.suc.suc], lam: R) {
    char_poly_det[R](Nat.0, a, lam) = R.0 implies
        (matrix2_00[R](Nat.0, a) - lam) * (matrix2_11[R](Nat.0, a) - lam) -
            matrix2_01[R](Nat.0, a) * matrix2_10[R](Nat.0, a) = R.0
} by {
    if char_poly_det[R](Nat.0, a, lam) = R.0 {
        char_poly_det_eq_entries[R](a, lam)
        char_poly_det[R](Nat.0, a, lam) = char_poly_2x2[R](Nat.0, a, lam)
        char_poly_2x2[R](Nat.0, a, lam) = (matrix2_00[R](Nat.0, a) - lam) * (matrix2_11[R](Nat.0, a) - lam) -
            matrix2_01[R](Nat.0, a) * matrix2_10[R](Nat.0, a)
        (matrix2_00[R](Nat.0, a) - lam) * (matrix2_11[R](Nat.0, a) - lam) -
            matrix2_01[R](Nat.0, a) * matrix2_10[R](Nat.0, a) = R.0
    }
}

/// A coordinate vector satisfying both rows of `(A - lam * I) v = 0` is an
/// eigenvector of `A` for `lam`.
theorem vec_solves_is_eigenvector[F: Field](
    a: Matrix[F, Nat.0.suc.suc, Nat.0.suc.suc], lam: F, x: F, y: F
) {
    (matrix2_00[F](Nat.0, a) - lam) * x + matrix2_01[F](Nat.0, a) * y = F.0 and
    matrix2_10[F](Nat.0, a) * x + (matrix2_11[F](Nat.0, a) - lam) * y = F.0 and
    is_nonzero_vector[F](Nat.0.suc.suc, matrix2_vec[F](Nat.0.suc.suc, x, y))
        implies is_eigenvector[F](Nat.0.suc.suc, a, lam, matrix2_vec[F](Nat.0.suc.suc, x, y))
} by {
    if (matrix2_00[F](Nat.0, a) - lam) * x + matrix2_01[F](Nat.0, a) * y = F.0 and
        matrix2_10[F](Nat.0, a) * x + (matrix2_11[F](Nat.0, a) - lam) * y = F.0 and
        is_nonzero_vector[F](Nat.0.suc.suc, matrix2_vec[F](Nat.0.suc.suc, x, y)) {
        forall(i: Fin[Nat.0.suc.suc]) {
            fin_two_index_cases(i)
            if i.value = Nat.0 {
                fin_two_zero_value_two
                i.value = fin_zero(Nat.0.suc).value
                ext(Nat.0.suc.suc, i, fin_zero(Nat.0.suc))
                i = fin_zero(Nat.0.suc)
                matrix_apply_B_vec_0[F](a, lam, x, y)
                matrix_apply[F](Nat.0.suc.suc,
                    matrix_sub[F](Nat.0.suc.suc, Nat.0.suc.suc, a,
                        matrix_smul[F, F](ring_as_module[F], Nat.0.suc.suc, Nat.0.suc.suc, lam, matrix_one[F](Nat.0.suc.suc))),
                    matrix2_vec[F](Nat.0.suc.suc, x, y), fin_zero(Nat.0.suc)) =
                    (matrix2_00[F](Nat.0, a) - lam) * x + matrix2_01[F](Nat.0, a) * y
                matrix_apply[F](Nat.0.suc.suc,
                    matrix_sub[F](Nat.0.suc.suc, Nat.0.suc.suc, a,
                        matrix_smul[F, F](ring_as_module[F], Nat.0.suc.suc, Nat.0.suc.suc, lam, matrix_one[F](Nat.0.suc.suc))),
                    matrix2_vec[F](Nat.0.suc.suc, x, y), fin_zero(Nat.0.suc)) = F.0
                matrix_apply[F](Nat.0.suc.suc,
                    matrix_sub[F](Nat.0.suc.suc, Nat.0.suc.suc, a,
                        matrix_smul[F, F](ring_as_module[F], Nat.0.suc.suc, Nat.0.suc.suc, lam, matrix_one[F](Nat.0.suc.suc))),
                    matrix2_vec[F](Nat.0.suc.suc, x, y), i) = F.0
            } else {
                fin_two_one_value_two
                i.value = fin_succ(Nat.0.suc, fin_zero(Nat.0)).value
                ext(Nat.0.suc.suc, i, fin_succ(Nat.0.suc, fin_zero(Nat.0)))
                i = fin_succ(Nat.0.suc, fin_zero(Nat.0))
                matrix_apply_B_vec_1[F](a, lam, x, y)
                matrix_apply[F](Nat.0.suc.suc,
                    matrix_sub[F](Nat.0.suc.suc, Nat.0.suc.suc, a,
                        matrix_smul[F, F](ring_as_module[F], Nat.0.suc.suc, Nat.0.suc.suc, lam, matrix_one[F](Nat.0.suc.suc))),
                    matrix2_vec[F](Nat.0.suc.suc, x, y), fin_succ(Nat.0.suc, fin_zero(Nat.0))) =
                    matrix2_10[F](Nat.0, a) * x + (matrix2_11[F](Nat.0, a) - lam) * y
                matrix_apply[F](Nat.0.suc.suc,
                    matrix_sub[F](Nat.0.suc.suc, Nat.0.suc.suc, a,
                        matrix_smul[F, F](ring_as_module[F], Nat.0.suc.suc, Nat.0.suc.suc, lam, matrix_one[F](Nat.0.suc.suc))),
                    matrix2_vec[F](Nat.0.suc.suc, x, y), fin_succ(Nat.0.suc, fin_zero(Nat.0))) = F.0
                matrix_apply[F](Nat.0.suc.suc,
                    matrix_sub[F](Nat.0.suc.suc, Nat.0.suc.suc, a,
                        matrix_smul[F, F](ring_as_module[F], Nat.0.suc.suc, Nat.0.suc.suc, lam, matrix_one[F](Nat.0.suc.suc))),
                    matrix2_vec[F](Nat.0.suc.suc, x, y), i) = F.0
            }
        }
        forall(i: Fin[Nat.0.suc.suc]) {
            matrix_apply_char_poly_sub[F](a, lam, matrix2_vec[F](Nat.0.suc.suc, x, y), i)
            matrix_apply[F](Nat.0.suc.suc,
                matrix_sub[F](Nat.0.suc.suc, Nat.0.suc.suc, a,
                    matrix_smul[F, F](ring_as_module[F], Nat.0.suc.suc, Nat.0.suc.suc, lam, matrix_one[F](Nat.0.suc.suc))),
                matrix2_vec[F](Nat.0.suc.suc, x, y), i) =
                matrix_apply[F](Nat.0.suc.suc, a, matrix2_vec[F](Nat.0.suc.suc, x, y), i) - lam * matrix2_vec[F](Nat.0.suc.suc, x, y, i)
            matrix_apply[F](Nat.0.suc.suc,
                matrix_sub[F](Nat.0.suc.suc, Nat.0.suc.suc, a,
                    matrix_smul[F, F](ring_as_module[F], Nat.0.suc.suc, Nat.0.suc.suc, lam, matrix_one[F](Nat.0.suc.suc))),
                matrix2_vec[F](Nat.0.suc.suc, x, y), i) = F.0
            matrix_apply[F](Nat.0.suc.suc, a, matrix2_vec[F](Nat.0.suc.suc, x, y), i) - lam * matrix2_vec[F](Nat.0.suc.suc, x, y, i) = F.0
            ring_sub_eq_zero_iff_eq[F](matrix_apply[F](Nat.0.suc.suc, a, matrix2_vec[F](Nat.0.suc.suc, x, y), i), lam * matrix2_vec[F](Nat.0.suc.suc, x, y, i))
            matrix_apply[F](Nat.0.suc.suc, a, matrix2_vec[F](Nat.0.suc.suc, x, y), i) = lam * matrix2_vec[F](Nat.0.suc.suc, x, y, i)
        }
        is_eigenvector_intro[F](Nat.0.suc.suc, a, lam, matrix2_vec[F](Nat.0.suc.suc, x, y))
        is_eigenvector[F](Nat.0.suc.suc, a, lam, matrix2_vec[F](Nat.0.suc.suc, x, y))
    }
}

/// If the upper-left entry of `A - lam * I` is nonzero and the determinant
/// vanishes, then `lam` is an eigenvalue.
theorem eigenvector_case_b00_nonzero[F: Field](a: Matrix[F, Nat.0.suc.suc, Nat.0.suc.suc], lam: F) {
    matrix2_00[F](Nat.0, a) - lam != F.0 and
    (matrix2_00[F](Nat.0, a) - lam) * (matrix2_11[F](Nat.0, a) - lam) -
        matrix2_01[F](Nat.0, a) * matrix2_10[F](Nat.0, a) = F.0
        implies is_eigenvalue[F](Nat.0.suc.suc, a, lam)
} by {
    if matrix2_00[F](Nat.0, a) - lam != F.0 and
        (matrix2_00[F](Nat.0, a) - lam) * (matrix2_11[F](Nat.0, a) - lam) -
            matrix2_01[F](Nat.0, a) * matrix2_10[F](Nat.0, a) = F.0 {
        mul_neg_right[F](matrix2_01[F](Nat.0, a), matrix2_00[F](Nat.0, a) - lam)
        matrix2_01[F](Nat.0, a) * (-(matrix2_00[F](Nat.0, a) - lam)) =
            -(matrix2_01[F](Nat.0, a) * (matrix2_00[F](Nat.0, a) - lam))
        ring_mul_comm_sub_zero[F](matrix2_00[F](Nat.0, a) - lam, matrix2_01[F](Nat.0, a))
        (matrix2_00[F](Nat.0, a) - lam) * matrix2_01[F](Nat.0, a) -
            matrix2_01[F](Nat.0, a) * (matrix2_00[F](Nat.0, a) - lam) = F.0
        (matrix2_00[F](Nat.0, a) - lam) * matrix2_01[F](Nat.0, a) +
            matrix2_01[F](Nat.0, a) * (-(matrix2_00[F](Nat.0, a) - lam)) = F.0
        mul_neg_right[F](matrix2_11[F](Nat.0, a) - lam, matrix2_00[F](Nat.0, a) - lam)
        (matrix2_11[F](Nat.0, a) - lam) * (-(matrix2_00[F](Nat.0, a) - lam)) =
            -((matrix2_11[F](Nat.0, a) - lam) * (matrix2_00[F](Nat.0, a) - lam))
        sub_neg_rev[F]((matrix2_00[F](Nat.0, a) - lam) * (matrix2_11[F](Nat.0, a) - lam),
            matrix2_01[F](Nat.0, a) * matrix2_10[F](Nat.0, a))
        matrix2_01[F](Nat.0, a) * matrix2_10[F](Nat.0, a) -
            (matrix2_00[F](Nat.0, a) - lam) * (matrix2_11[F](Nat.0, a) - lam) =
            -((matrix2_00[F](Nat.0, a) - lam) * (matrix2_11[F](Nat.0, a) - lam) -
                matrix2_01[F](Nat.0, a) * matrix2_10[F](Nat.0, a))
        matrix2_10[F](Nat.0, a) * matrix2_01[F](Nat.0, a) -
            (matrix2_00[F](Nat.0, a) - lam) * (matrix2_11[F](Nat.0, a) - lam) = F.0
        matrix2_10[F](Nat.0, a) * matrix2_01[F](Nat.0, a) +
            (matrix2_11[F](Nat.0, a) - lam) * (-(matrix2_00[F](Nat.0, a) - lam)) = F.0
        ring_neg_eq_zero_iff[F](matrix2_00[F](Nat.0, a) - lam)
        (-(matrix2_00[F](Nat.0, a) - lam) = F.0) = (matrix2_00[F](Nat.0, a) - lam = F.0)
        if -(matrix2_00[F](Nat.0, a) - lam) = F.0 {
            matrix2_00[F](Nat.0, a) - lam = F.0
            false
        }
        -(matrix2_00[F](Nat.0, a) - lam) != F.0
        matrix2_vec_nonzero_1[F](matrix2_01[F](Nat.0, a), -(matrix2_00[F](Nat.0, a) - lam))
        is_nonzero_vector[F](Nat.0.suc.suc, matrix2_vec[F](Nat.0.suc.suc,
            matrix2_01[F](Nat.0, a), -(matrix2_00[F](Nat.0, a) - lam)))
        vec_solves_is_eigenvector[F](a, lam, matrix2_01[F](Nat.0, a), -(matrix2_00[F](Nat.0, a) - lam))
        is_eigenvector[F](Nat.0.suc.suc, a, lam, matrix2_vec[F](Nat.0.suc.suc,
            matrix2_01[F](Nat.0, a), -(matrix2_00[F](Nat.0, a) - lam)))
        is_eigenvalue_intro[F](Nat.0.suc.suc, a, lam, matrix2_vec[F](Nat.0.suc.suc,
            matrix2_01[F](Nat.0, a), -(matrix2_00[F](Nat.0, a) - lam)))
        is_eigenvalue[F](Nat.0.suc.suc, a, lam)
    }
}

/// If the first row of `A - lam * I` vanishes and the lower-left entry is
/// nonzero, then `lam` is an eigenvalue.
theorem eigenvector_case_b00_zero_b10_nonzero[F: Field](a: Matrix[F, Nat.0.suc.suc, Nat.0.suc.suc], lam: F) {
    matrix2_00[F](Nat.0, a) - lam = F.0 and matrix2_10[F](Nat.0, a) != F.0 and
    (matrix2_00[F](Nat.0, a) - lam) * (matrix2_11[F](Nat.0, a) - lam) -
        matrix2_01[F](Nat.0, a) * matrix2_10[F](Nat.0, a) = F.0
        implies is_eigenvalue[F](Nat.0.suc.suc, a, lam)
} by {
    if matrix2_00[F](Nat.0, a) - lam = F.0 and matrix2_10[F](Nat.0, a) != F.0 and
        (matrix2_00[F](Nat.0, a) - lam) * (matrix2_11[F](Nat.0, a) - lam) -
            matrix2_01[F](Nat.0, a) * matrix2_10[F](Nat.0, a) = F.0 {
        mul_zero_left[F](matrix2_11[F](Nat.0, a) - lam)
        F.0 * (matrix2_11[F](Nat.0, a) - lam) = F.0
        (matrix2_00[F](Nat.0, a) - lam) * (matrix2_11[F](Nat.0, a) - lam) = F.0
        F.0 - matrix2_01[F](Nat.0, a) * matrix2_10[F](Nat.0, a) = F.0
        -(matrix2_01[F](Nat.0, a) * matrix2_10[F](Nat.0, a)) = F.0
        ring_neg_eq_zero_iff[F](matrix2_01[F](Nat.0, a) * matrix2_10[F](Nat.0, a))
        matrix2_01[F](Nat.0, a) * matrix2_10[F](Nat.0, a) = F.0
        field_mul_eq_zero[F](matrix2_01[F](Nat.0, a), matrix2_10[F](Nat.0, a))
        matrix2_01[F](Nat.0, a) = F.0 or matrix2_10[F](Nat.0, a) = F.0
        if matrix2_01[F](Nat.0, a) = F.0 {
            matrix2_01[F](Nat.0, a) = F.0
        } else {
            matrix2_10[F](Nat.0, a) = F.0
            false
        }
        matrix2_01[F](Nat.0, a) = F.0
        mul_zero_left[F](matrix2_01[F](Nat.0, a))
        F.0 * matrix2_01[F](Nat.0, a) = F.0
        mul_zero_left[F](matrix2_11[F](Nat.0, a) - lam)
        F.0 * (matrix2_11[F](Nat.0, a) - lam) = F.0
        mul_neg_right[F](matrix2_11[F](Nat.0, a) - lam, matrix2_10[F](Nat.0, a))
        (matrix2_11[F](Nat.0, a) - lam) * (-matrix2_10[F](Nat.0, a)) =
            -((matrix2_11[F](Nat.0, a) - lam) * matrix2_10[F](Nat.0, a))
        ring_mul_comm_sub_zero[F](matrix2_10[F](Nat.0, a), matrix2_11[F](Nat.0, a) - lam)
        matrix2_10[F](Nat.0, a) * (matrix2_11[F](Nat.0, a) - lam) -
            (matrix2_11[F](Nat.0, a) - lam) * matrix2_10[F](Nat.0, a) = F.0
        matrix2_10[F](Nat.0, a) * (matrix2_11[F](Nat.0, a) - lam) +
            (matrix2_11[F](Nat.0, a) - lam) * (-matrix2_10[F](Nat.0, a)) = F.0
        (matrix2_00[F](Nat.0, a) - lam) * (matrix2_11[F](Nat.0, a) - lam) +
            matrix2_01[F](Nat.0, a) * (-matrix2_10[F](Nat.0, a)) = F.0
        ring_neg_eq_zero_iff[F](matrix2_10[F](Nat.0, a))
        (-matrix2_10[F](Nat.0, a) = F.0) = (matrix2_10[F](Nat.0, a) = F.0)
        if -matrix2_10[F](Nat.0, a) = F.0 {
            matrix2_10[F](Nat.0, a) = F.0
            false
        }
        -matrix2_10[F](Nat.0, a) != F.0
        matrix2_vec_nonzero_1[F](matrix2_11[F](Nat.0, a) - lam, -matrix2_10[F](Nat.0, a))
        is_nonzero_vector[F](Nat.0.suc.suc, matrix2_vec[F](Nat.0.suc.suc,
            matrix2_11[F](Nat.0, a) - lam, -matrix2_10[F](Nat.0, a)))
        vec_solves_is_eigenvector[F](a, lam, matrix2_11[F](Nat.0, a) - lam, -matrix2_10[F](Nat.0, a))
        is_eigenvector[F](Nat.0.suc.suc, a, lam, matrix2_vec[F](Nat.0.suc.suc,
            matrix2_11[F](Nat.0, a) - lam, -matrix2_10[F](Nat.0, a)))
        is_eigenvalue_intro[F](Nat.0.suc.suc, a, lam, matrix2_vec[F](Nat.0.suc.suc,
            matrix2_11[F](Nat.0, a) - lam, -matrix2_10[F](Nat.0, a)))
        is_eigenvalue[F](Nat.0.suc.suc, a, lam)
    }
}

/// If the first row and the lower-left entry of `A - lam * I` vanish, then
/// `lam` is an eigenvalue.
theorem eigenvector_case_b00_zero_b10_zero[F: Field](a: Matrix[F, Nat.0.suc.suc, Nat.0.suc.suc], lam: F) {
    matrix2_00[F](Nat.0, a) - lam = F.0 and matrix2_10[F](Nat.0, a) = F.0 and F.1 != F.0
        implies is_eigenvalue[F](Nat.0.suc.suc, a, lam)
} by {
    if matrix2_00[F](Nat.0, a) - lam = F.0 and matrix2_10[F](Nat.0, a) = F.0 and F.1 != F.0 {
        mul_zero_right[F](matrix2_01[F](Nat.0, a))
        matrix2_01[F](Nat.0, a) * F.0 = F.0
        mul_zero_left[F](matrix2_01[F](Nat.0, a))
        F.0 * matrix2_01[F](Nat.0, a) = F.0
        mul_zero_left[F](matrix2_11[F](Nat.0, a) - lam)
        F.0 * (matrix2_11[F](Nat.0, a) - lam) = F.0
        mul_zero_left[F](F.1)
        F.0 * F.1 = F.0
        (matrix2_00[F](Nat.0, a) - lam) * F.1 + matrix2_01[F](Nat.0, a) * F.0 = F.0
        matrix2_10[F](Nat.0, a) * F.1 + (matrix2_11[F](Nat.0, a) - lam) * F.0 = F.0
        matrix2_vec_nonzero_0[F](F.1, F.0)
        is_nonzero_vector[F](Nat.0.suc.suc, matrix2_vec[F](Nat.0.suc.suc, F.1, F.0))
        vec_solves_is_eigenvector[F](a, lam, F.1, F.0)
        is_eigenvector[F](Nat.0.suc.suc, a, lam, matrix2_vec[F](Nat.0.suc.suc, F.1, F.0))
        is_eigenvalue_intro[F](Nat.0.suc.suc, a, lam, matrix2_vec[F](Nat.0.suc.suc, F.1, F.0))
        is_eigenvalue[F](Nat.0.suc.suc, a, lam)
    }
}

/// A vanishing characteristic polynomial of a two-by-two matrix over a field
/// with `1 != 0` makes `lam` an eigenvalue.
theorem char_poly_zero_implies_eigenvalue[F: Field](a: Matrix[F, Nat.0.suc.suc, Nat.0.suc.suc], lam: F) {
    char_poly_det[F](Nat.0, a, lam) = F.0 and F.1 != F.0
        implies is_eigenvalue[F](Nat.0.suc.suc, a, lam)
} by {
    if char_poly_det[F](Nat.0, a, lam) = F.0 and F.1 != F.0 {
        char_poly_det_entry_zero[F](a, lam)
        (matrix2_00[F](Nat.0, a) - lam) * (matrix2_11[F](Nat.0, a) - lam) -
            matrix2_01[F](Nat.0, a) * matrix2_10[F](Nat.0, a) = F.0
        if matrix2_00[F](Nat.0, a) - lam = F.0 {
            mul_zero_left[F](matrix2_11[F](Nat.0, a) - lam)
            F.0 * (matrix2_11[F](Nat.0, a) - lam) = F.0
            (matrix2_00[F](Nat.0, a) - lam) * (matrix2_11[F](Nat.0, a) - lam) = F.0
            F.0 - matrix2_01[F](Nat.0, a) * matrix2_10[F](Nat.0, a) = F.0
            -(matrix2_01[F](Nat.0, a) * matrix2_10[F](Nat.0, a)) = F.0
            ring_neg_eq_zero_iff[F](matrix2_01[F](Nat.0, a) * matrix2_10[F](Nat.0, a))
            matrix2_01[F](Nat.0, a) * matrix2_10[F](Nat.0, a) = F.0
            field_mul_eq_zero[F](matrix2_01[F](Nat.0, a), matrix2_10[F](Nat.0, a))
            matrix2_01[F](Nat.0, a) = F.0 or matrix2_10[F](Nat.0, a) = F.0
            if matrix2_10[F](Nat.0, a) = F.0 {
                eigenvector_case_b00_zero_b10_zero[F](a, lam)
                is_eigenvalue[F](Nat.0.suc.suc, a, lam)
            } else {
                eigenvector_case_b00_zero_b10_nonzero[F](a, lam)
                is_eigenvalue[F](Nat.0.suc.suc, a, lam)
            }
            is_eigenvalue[F](Nat.0.suc.suc, a, lam)
        } else {
            eigenvector_case_b00_nonzero[F](a, lam)
            is_eigenvalue[F](Nat.0.suc.suc, a, lam)
        }
        is_eigenvalue[F](Nat.0.suc.suc, a, lam)
    }
}

/// `lam` is an eigenvalue of a two-by-two matrix over a field exactly when
/// the characteristic polynomial vanishes at `lam`.
theorem eigenvalue_iff_char_poly_zero[F: Field](a: Matrix[F, Nat.0.suc.suc, Nat.0.suc.suc], lam: F) {
    F.1 != F.0 implies
        (is_eigenvalue[F](Nat.0.suc.suc, a, lam)) = (char_poly_det[F](Nat.0, a, lam) = F.0)
} by {
    if F.1 != F.0 {
        if is_eigenvalue[F](Nat.0.suc.suc, a, lam) {
            eigenvalue_implies_char_poly_zero[F](a, lam)
            char_poly_det[F](Nat.0, a, lam) = F.0
        }
        if char_poly_det[F](Nat.0, a, lam) = F.0 {
            char_poly_zero_implies_eigenvalue[F](a, lam)
            is_eigenvalue[F](Nat.0.suc.suc, a, lam)
        }
        (is_eigenvalue[F](Nat.0.suc.suc, a, lam)) = (char_poly_det[F](Nat.0, a, lam) = F.0)
    }
}

/// True if a square matrix of size `n + 2` has a two-sided inverse.
define is_invertible_two[R: Ring](n: Nat, a: Matrix[R, n.suc.suc, n.suc.suc]) -> Bool {
    exists(b: Matrix[R, n.suc.suc, n.suc.suc]) {
        square_matrix_mul[R](n.suc.suc, a, b) = matrix_one[R](n.suc.suc) and
        square_matrix_mul[R](n.suc.suc, b, a) = matrix_one[R](n.suc.suc)
    }
}

/// A vanishing characteristic polynomial makes `A - lam * I` singular.
theorem char_poly_zero_implies_not_invertible[F: Field](a: Matrix[F, Nat.0.suc.suc, Nat.0.suc.suc], lam: F) {
    char_poly_det[F](Nat.0, a, lam) = F.0 implies
        not is_invertible_two[F](Nat.0,
            matrix_sub[F](Nat.0.suc.suc, Nat.0.suc.suc, a,
                matrix_smul[F, F](ring_as_module[F], Nat.0.suc.suc, Nat.0.suc.suc, lam, matrix_one[F](Nat.0.suc.suc))))
} by {
    if char_poly_det[F](Nat.0, a, lam) = F.0 {
        char_poly_det_eq_matrix2_det[F](a, lam)
        char_poly_det[F](Nat.0, a, lam) = matrix2_det[F](Nat.0,
            matrix_sub[F](Nat.0.suc.suc, Nat.0.suc.suc, a,
                matrix_smul[F, F](ring_as_module[F], Nat.0.suc.suc, Nat.0.suc.suc, lam, matrix_one[F](Nat.0.suc.suc))))
        matrix2_det[F](Nat.0,
            matrix_sub[F](Nat.0.suc.suc, Nat.0.suc.suc, a,
                matrix_smul[F, F](ring_as_module[F], Nat.0.suc.suc, Nat.0.suc.suc, lam, matrix_one[F](Nat.0.suc.suc)))) = F.0
        if is_invertible_two[F](Nat.0,
            matrix_sub[F](Nat.0.suc.suc, Nat.0.suc.suc, a,
                matrix_smul[F, F](ring_as_module[F], Nat.0.suc.suc, Nat.0.suc.suc, lam, matrix_one[F](Nat.0.suc.suc)))) {
            matrix2_invertible_iff_det_nonzero[F](
                matrix_sub[F](Nat.0.suc.suc, Nat.0.suc.suc, a,
                    matrix_smul[F, F](ring_as_module[F], Nat.0.suc.suc, Nat.0.suc.suc, lam, matrix_one[F](Nat.0.suc.suc))))
            matrix2_det[F](Nat.0,
                matrix_sub[F](Nat.0.suc.suc, Nat.0.suc.suc, a,
                    matrix_smul[F, F](ring_as_module[F], Nat.0.suc.suc, Nat.0.suc.suc, lam, matrix_one[F](Nat.0.suc.suc)))) != F.0
            false
        }
        not is_invertible_two[F](Nat.0,
            matrix_sub[F](Nat.0.suc.suc, Nat.0.suc.suc, a,
                matrix_smul[F, F](ring_as_module[F], Nat.0.suc.suc, Nat.0.suc.suc, lam, matrix_one[F](Nat.0.suc.suc))))
    }
}

/// A singular `A - lam * I` makes the characteristic polynomial vanish.
theorem not_invertible_implies_char_poly_zero[F: Field](a: Matrix[F, Nat.0.suc.suc, Nat.0.suc.suc], lam: F) {
    (not is_invertible_two[F](Nat.0,
        matrix_sub[F](Nat.0.suc.suc, Nat.0.suc.suc, a,
            matrix_smul[F, F](ring_as_module[F], Nat.0.suc.suc, Nat.0.suc.suc, lam, matrix_one[F](Nat.0.suc.suc)))))
        implies char_poly_det[F](Nat.0, a, lam) = F.0
} by {
    if not is_invertible_two[F](Nat.0,
        matrix_sub[F](Nat.0.suc.suc, Nat.0.suc.suc, a,
            matrix_smul[F, F](ring_as_module[F], Nat.0.suc.suc, Nat.0.suc.suc, lam, matrix_one[F](Nat.0.suc.suc)))) {
        matrix2_invertible_iff_det_nonzero[F](
            matrix_sub[F](Nat.0.suc.suc, Nat.0.suc.suc, a,
                matrix_smul[F, F](ring_as_module[F], Nat.0.suc.suc, Nat.0.suc.suc, lam, matrix_one[F](Nat.0.suc.suc))))
        if matrix2_det[F](Nat.0,
            matrix_sub[F](Nat.0.suc.suc, Nat.0.suc.suc, a,
                matrix_smul[F, F](ring_as_module[F], Nat.0.suc.suc, Nat.0.suc.suc, lam, matrix_one[F](Nat.0.suc.suc)))) != F.0 {
            is_invertible_two[F](Nat.0,
                matrix_sub[F](Nat.0.suc.suc, Nat.0.suc.suc, a,
                    matrix_smul[F, F](ring_as_module[F], Nat.0.suc.suc, Nat.0.suc.suc, lam, matrix_one[F](Nat.0.suc.suc))))
            false
        }
        matrix2_det[F](Nat.0,
            matrix_sub[F](Nat.0.suc.suc, Nat.0.suc.suc, a,
                matrix_smul[F, F](ring_as_module[F], Nat.0.suc.suc, Nat.0.suc.suc, lam, matrix_one[F](Nat.0.suc.suc)))) = F.0
        char_poly_det_eq_matrix2_det[F](a, lam)
        char_poly_det[F](Nat.0, a, lam) = matrix2_det[F](Nat.0,
            matrix_sub[F](Nat.0.suc.suc, Nat.0.suc.suc, a,
                matrix_smul[F, F](ring_as_module[F], Nat.0.suc.suc, Nat.0.suc.suc, lam, matrix_one[F](Nat.0.suc.suc))))
        char_poly_det[F](Nat.0, a, lam) = F.0
    }
}

/// The characteristic polynomial of `A` vanishes at `lam` exactly when
/// `A - lam * I` is not invertible.
theorem char_poly_zero_iff_not_invertible[F: Field](a: Matrix[F, Nat.0.suc.suc, Nat.0.suc.suc], lam: F) {
    (char_poly_det[F](Nat.0, a, lam) = F.0) =
        (not is_invertible_two[F](Nat.0,
            matrix_sub[F](Nat.0.suc.suc, Nat.0.suc.suc, a,
                matrix_smul[F, F](ring_as_module[F], Nat.0.suc.suc, Nat.0.suc.suc, lam, matrix_one[F](Nat.0.suc.suc)))))
} by {
    char_poly_zero_implies_not_invertible[F](a, lam)
    not_invertible_implies_char_poly_zero[F](a, lam)
    if char_poly_det[F](Nat.0, a, lam) = F.0 {
        not is_invertible_two[F](Nat.0,
            matrix_sub[F](Nat.0.suc.suc, Nat.0.suc.suc, a,
                matrix_smul[F, F](ring_as_module[F], Nat.0.suc.suc, Nat.0.suc.suc, lam, matrix_one[F](Nat.0.suc.suc))))
    }
    if not is_invertible_two[F](Nat.0,
        matrix_sub[F](Nat.0.suc.suc, Nat.0.suc.suc, a,
            matrix_smul[F, F](ring_as_module[F], Nat.0.suc.suc, Nat.0.suc.suc, lam, matrix_one[F](Nat.0.suc.suc)))) {
        char_poly_det[F](Nat.0, a, lam) = F.0
    }
    (char_poly_det[F](Nat.0, a, lam) = F.0) =
        (not is_invertible_two[F](Nat.0,
            matrix_sub[F](Nat.0.suc.suc, Nat.0.suc.suc, a,
                matrix_smul[F, F](ring_as_module[F], Nat.0.suc.suc, Nat.0.suc.suc, lam, matrix_one[F](Nat.0.suc.suc)))))
}

// ---------------------------------------------------------------------------
// The trace of a square matrix
// ---------------------------------------------------------------------------

/// The `i`th diagonal entry of a square matrix.
define matrix_trace_term[R: Ring](n: Nat, a: Matrix[R, n, n], i: Fin[n]) -> R {
    matrix_entry[R](n, n, a, i, i)
}

/// The trace of a square matrix, the sum of its diagonal entries.
define matrix_trace[R: Ring](n: Nat, a: Matrix[R, n, n]) -> R {
    fin_sum[R](n, matrix_trace_term[R](n, a))
}

/// The trace of a two-by-two matrix is the sum of the two diagonal entries.
theorem matrix_trace_two[R: Ring](a: Matrix[R, Nat.0.suc.suc, Nat.0.suc.suc]) {
    matrix_trace[R](Nat.0.suc.suc, a) = matrix2_00[R](Nat.0, a) + matrix2_11[R](Nat.0, a)
} by {
    matrix_trace[R](Nat.0.suc.suc, a) = fin_sum[R](Nat.0.suc.suc, matrix_trace_term[R](Nat.0.suc.suc, a))
    fin_sum_two[R](matrix_trace_term[R](Nat.0.suc.suc, a))
    fin_sum[R](Nat.0.suc.suc, matrix_trace_term[R](Nat.0.suc.suc, a)) =
        matrix_trace_term[R](Nat.0.suc.suc, a, fin_zero(Nat.0.suc)) +
        matrix_trace_term[R](Nat.0.suc.suc, a, fin_succ(Nat.0.suc, fin_zero(Nat.0)))
    matrix_trace_term[R](Nat.0.suc.suc, a, fin_zero(Nat.0.suc)) =
        matrix_entry[R](Nat.0.suc.suc, Nat.0.suc.suc, a, fin_zero(Nat.0.suc), fin_zero(Nat.0.suc))
    matrix_trace_term[R](Nat.0.suc.suc, a, fin_succ(Nat.0.suc, fin_zero(Nat.0))) =
        matrix_entry[R](Nat.0.suc.suc, Nat.0.suc.suc, a, fin_succ(Nat.0.suc, fin_zero(Nat.0)), fin_succ(Nat.0.suc, fin_zero(Nat.0)))
    matrix2_00[R](Nat.0, a) = matrix_entry[R](Nat.0.suc.suc, Nat.0.suc.suc, a, fin_zero(Nat.0.suc), fin_zero(Nat.0.suc))
    matrix2_11[R](Nat.0, a) = matrix_entry[R](Nat.0.suc.suc, Nat.0.suc.suc, a, fin_succ(Nat.0.suc, fin_zero(Nat.0)), fin_succ(Nat.0.suc, fin_zero(Nat.0)))
    matrix_trace[R](Nat.0.suc.suc, a) = matrix2_00[R](Nat.0, a) + matrix2_11[R](Nat.0, a)
}

/// The trace of a two-by-two matrix agrees with the entrywise trace.
theorem matrix_trace_two_eq_matrix2_trace[R: Ring](a: Matrix[R, Nat.0.suc.suc, Nat.0.suc.suc]) {
    matrix_trace[R](Nat.0.suc.suc, a) = matrix2_trace[R](Nat.0, a)
} by {
    matrix_trace_two[R](a)
    matrix_trace[R](Nat.0.suc.suc, a) = matrix2_00[R](Nat.0, a) + matrix2_11[R](Nat.0, a)
    matrix2_trace[R](Nat.0, a) = matrix2_00[R](Nat.0, a) + matrix2_11[R](Nat.0, a)
    matrix_trace[R](Nat.0.suc.suc, a) = matrix2_trace[R](Nat.0, a)
}

/// The trace of a transpose is the trace of the matrix.
theorem matrix_trace_transpose[R: Ring](n: Nat, a: Matrix[R, n, n]) {
    matrix_trace[R](n, matrix_transpose[R](n, n, a)) = matrix_trace[R](n, a)
} by {
    forall(i: Fin[n]) {
        matrix_entry_transpose[R](n, n, a, i, i)
        matrix_entry[R](n, n, matrix_transpose[R](n, n, a), i, i) = matrix_entry[R](n, n, a, i, i)
        matrix_trace_term[R](n, matrix_transpose[R](n, n, a), i) = matrix_entry[R](n, n, matrix_transpose[R](n, n, a), i, i)
        matrix_trace_term[R](n, a, i) = matrix_entry[R](n, n, a, i, i)
        matrix_trace_term[R](n, matrix_transpose[R](n, n, a), i) = matrix_trace_term[R](n, a, i)
    }
    fin_sum_pointwise_eq[R](n, matrix_trace_term[R](n, matrix_transpose[R](n, n, a)), matrix_trace_term[R](n, a))
    fin_sum[R](n, matrix_trace_term[R](n, matrix_transpose[R](n, n, a))) = fin_sum[R](n, matrix_trace_term[R](n, a))
    matrix_trace[R](n, matrix_transpose[R](n, n, a)) = fin_sum[R](n, matrix_trace_term[R](n, matrix_transpose[R](n, n, a)))
    matrix_trace[R](n, a) = fin_sum[R](n, matrix_trace_term[R](n, a))
    matrix_trace[R](n, matrix_transpose[R](n, n, a)) = matrix_trace[R](n, a)
}

/// The trace of a sum is the sum of the traces.
theorem matrix_trace_add[R: Ring](n: Nat, a: Matrix[R, n, n], b: Matrix[R, n, n]) {
    matrix_trace[R](n, matrix_add[R](n, n, a, b)) = matrix_trace[R](n, a) + matrix_trace[R](n, b)
} by {
    forall(i: Fin[n]) {
        matrix_entry_add[R](n, n, a, b, i, i)
        matrix_entry[R](n, n, matrix_add[R](n, n, a, b), i, i) = matrix_entry[R](n, n, a, i, i) + matrix_entry[R](n, n, b, i, i)
        matrix_trace_term[R](n, matrix_add[R](n, n, a, b), i) = matrix_entry[R](n, n, matrix_add[R](n, n, a, b), i, i)
        matrix_trace_term[R](n, a, i) = matrix_entry[R](n, n, a, i, i)
        matrix_trace_term[R](n, b, i) = matrix_entry[R](n, n, b, i, i)
        matrix_trace_term[R](n, matrix_add[R](n, n, a, b), i) = matrix_trace_term[R](n, a, i) + matrix_trace_term[R](n, b, i)
        fin_pointwise_add[R](n, matrix_trace_term[R](n, a), matrix_trace_term[R](n, b), i) =
            matrix_trace_term[R](n, a, i) + matrix_trace_term[R](n, b, i)
        matrix_trace_term[R](n, matrix_add[R](n, n, a, b), i) =
            fin_pointwise_add[R](n, matrix_trace_term[R](n, a), matrix_trace_term[R](n, b), i)
    }
    fin_sum_pointwise_eq[R](n, matrix_trace_term[R](n, matrix_add[R](n, n, a, b)),
        fin_pointwise_add[R](n, matrix_trace_term[R](n, a), matrix_trace_term[R](n, b)))
    fin_sum[R](n, matrix_trace_term[R](n, matrix_add[R](n, n, a, b))) =
        fin_sum[R](n, fin_pointwise_add[R](n, matrix_trace_term[R](n, a), matrix_trace_term[R](n, b)))
    fin_sum_add[R](n, matrix_trace_term[R](n, a), matrix_trace_term[R](n, b))
    fin_sum[R](n, fin_pointwise_add[R](n, matrix_trace_term[R](n, a), matrix_trace_term[R](n, b))) =
        fin_sum[R](n, matrix_trace_term[R](n, a)) + fin_sum[R](n, matrix_trace_term[R](n, b))
    matrix_trace[R](n, matrix_add[R](n, n, a, b)) = fin_sum[R](n, matrix_trace_term[R](n, matrix_add[R](n, n, a, b)))
    matrix_trace[R](n, a) = fin_sum[R](n, matrix_trace_term[R](n, a))
    matrix_trace[R](n, b) = fin_sum[R](n, matrix_trace_term[R](n, b))
    matrix_trace[R](n, matrix_add[R](n, n, a, b)) = matrix_trace[R](n, a) + matrix_trace[R](n, b)
}

/// The trace of a scalar multiple is the scalar multiple of the trace.
theorem matrix_trace_smul[R: CommRing](n: Nat, r: R, a: Matrix[R, n, n]) {
    matrix_trace[R](n, matrix_smul[R, R](ring_as_module[R], n, n, r, a)) = r * matrix_trace[R](n, a)
} by {
    forall(i: Fin[n]) {
        matrix_entry_smul[R, R](ring_as_module[R], n, n, r, a, i, i)
        matrix_entry[R](n, n, matrix_smul[R, R](ring_as_module[R], n, n, r, a), i, i) =
            ring_as_module[R].smul(r, matrix_entry[R](n, n, a, i, i))
        ring_as_module_smul[R](r, matrix_entry[R](n, n, a, i, i))
        matrix_trace_term[R](n, matrix_smul[R, R](ring_as_module[R], n, n, r, a), i) =
            matrix_entry[R](n, n, matrix_smul[R, R](ring_as_module[R], n, n, r, a), i, i)
        matrix_trace_term[R](n, a, i) = matrix_entry[R](n, n, a, i, i)
        matrix_trace_term[R](n, matrix_smul[R, R](ring_as_module[R], n, n, r, a), i) = r * matrix_trace_term[R](n, a, i)
        fin_scale[R](n, r, matrix_trace_term[R](n, a), i) = r * matrix_trace_term[R](n, a, i)
        matrix_trace_term[R](n, matrix_smul[R, R](ring_as_module[R], n, n, r, a), i) =
            fin_scale[R](n, r, matrix_trace_term[R](n, a), i)
    }
    fin_sum_pointwise_eq[R](n, matrix_trace_term[R](n, matrix_smul[R, R](ring_as_module[R], n, n, r, a)),
        fin_scale[R](n, r, matrix_trace_term[R](n, a)))
    fin_sum[R](n, matrix_trace_term[R](n, matrix_smul[R, R](ring_as_module[R], n, n, r, a))) =
        fin_sum[R](n, fin_scale[R](n, r, matrix_trace_term[R](n, a)))
    fin_sum_scalar_mul[R](n, r, matrix_trace_term[R](n, a))
    fin_sum[R](n, fin_scale[R](n, r, matrix_trace_term[R](n, a))) = r * fin_sum[R](n, matrix_trace_term[R](n, a))
    matrix_trace[R](n, matrix_smul[R, R](ring_as_module[R], n, n, r, a)) =
        fin_sum[R](n, matrix_trace_term[R](n, matrix_smul[R, R](ring_as_module[R], n, n, r, a)))
    matrix_trace[R](n, a) = fin_sum[R](n, matrix_trace_term[R](n, a))
    matrix_trace[R](n, matrix_smul[R, R](ring_as_module[R], n, n, r, a)) = r * matrix_trace[R](n, a)
}

/// The trace of a product of two-by-two matrices is symmetric in the factors.
theorem matrix_trace_mul_comm_two[R: CommRing](a: Matrix[R, Nat.0.suc.suc, Nat.0.suc.suc], b: Matrix[R, Nat.0.suc.suc, Nat.0.suc.suc]) {
    matrix_trace[R](Nat.0.suc.suc, square_matrix_mul[R](Nat.0.suc.suc, a, b)) =
        matrix_trace[R](Nat.0.suc.suc, square_matrix_mul[R](Nat.0.suc.suc, b, a))
} by {
    matrix_trace_two[R](square_matrix_mul[R](Nat.0.suc.suc, a, b))
    matrix_trace[R](Nat.0.suc.suc, square_matrix_mul[R](Nat.0.suc.suc, a, b)) =
        matrix2_00[R](Nat.0, square_matrix_mul[R](Nat.0.suc.suc, a, b)) + matrix2_11[R](Nat.0, square_matrix_mul[R](Nat.0.suc.suc, a, b))
    matrix2_mul_entry_00[R](a, b)
    matrix2_mul_entry_11[R](a, b)
    matrix2_00[R](Nat.0, square_matrix_mul[R](Nat.0.suc.suc, a, b)) =
        matrix2_00[R](Nat.0, a) * matrix2_00[R](Nat.0, b) + matrix2_01[R](Nat.0, a) * matrix2_10[R](Nat.0, b)
    matrix2_11[R](Nat.0, square_matrix_mul[R](Nat.0.suc.suc, a, b)) =
        matrix2_10[R](Nat.0, a) * matrix2_01[R](Nat.0, b) + matrix2_11[R](Nat.0, a) * matrix2_11[R](Nat.0, b)
    matrix_trace[R](Nat.0.suc.suc, square_matrix_mul[R](Nat.0.suc.suc, a, b)) =
        (matrix2_00[R](Nat.0, a) * matrix2_00[R](Nat.0, b) + matrix2_01[R](Nat.0, a) * matrix2_10[R](Nat.0, b)) +
        (matrix2_10[R](Nat.0, a) * matrix2_01[R](Nat.0, b) + matrix2_11[R](Nat.0, a) * matrix2_11[R](Nat.0, b))
    matrix_trace_two[R](square_matrix_mul[R](Nat.0.suc.suc, b, a))
    matrix_trace[R](Nat.0.suc.suc, square_matrix_mul[R](Nat.0.suc.suc, b, a)) =
        matrix2_00[R](Nat.0, square_matrix_mul[R](Nat.0.suc.suc, b, a)) + matrix2_11[R](Nat.0, square_matrix_mul[R](Nat.0.suc.suc, b, a))
    matrix2_mul_entry_00[R](b, a)
    matrix2_mul_entry_11[R](b, a)
    matrix2_00[R](Nat.0, square_matrix_mul[R](Nat.0.suc.suc, b, a)) =
        matrix2_00[R](Nat.0, b) * matrix2_00[R](Nat.0, a) + matrix2_01[R](Nat.0, b) * matrix2_10[R](Nat.0, a)
    matrix2_11[R](Nat.0, square_matrix_mul[R](Nat.0.suc.suc, b, a)) =
        matrix2_10[R](Nat.0, b) * matrix2_01[R](Nat.0, a) + matrix2_11[R](Nat.0, b) * matrix2_11[R](Nat.0, a)
    matrix_trace[R](Nat.0.suc.suc, square_matrix_mul[R](Nat.0.suc.suc, b, a)) =
        (matrix2_00[R](Nat.0, b) * matrix2_00[R](Nat.0, a) + matrix2_01[R](Nat.0, b) * matrix2_10[R](Nat.0, a)) +
        (matrix2_10[R](Nat.0, b) * matrix2_01[R](Nat.0, a) + matrix2_11[R](Nat.0, b) * matrix2_11[R](Nat.0, a))
    matrix2_00[R](Nat.0, b) * matrix2_00[R](Nat.0, a) = matrix2_00[R](Nat.0, a) * matrix2_00[R](Nat.0, b)
    matrix2_01[R](Nat.0, b) * matrix2_10[R](Nat.0, a) = matrix2_10[R](Nat.0, a) * matrix2_01[R](Nat.0, b)
    matrix2_10[R](Nat.0, b) * matrix2_01[R](Nat.0, a) = matrix2_01[R](Nat.0, a) * matrix2_10[R](Nat.0, b)
    matrix2_11[R](Nat.0, b) * matrix2_11[R](Nat.0, a) = matrix2_11[R](Nat.0, a) * matrix2_11[R](Nat.0, b)
    matrix_trace[R](Nat.0.suc.suc, square_matrix_mul[R](Nat.0.suc.suc, b, a)) =
        (matrix2_00[R](Nat.0, a) * matrix2_00[R](Nat.0, b) + matrix2_10[R](Nat.0, a) * matrix2_01[R](Nat.0, b)) +
        (matrix2_01[R](Nat.0, a) * matrix2_10[R](Nat.0, b) + matrix2_11[R](Nat.0, a) * matrix2_11[R](Nat.0, b))
    add_swap_inner[R](matrix2_00[R](Nat.0, a) * matrix2_00[R](Nat.0, b),
        matrix2_10[R](Nat.0, a) * matrix2_01[R](Nat.0, b),
        matrix2_01[R](Nat.0, a) * matrix2_10[R](Nat.0, b),
        matrix2_11[R](Nat.0, a) * matrix2_11[R](Nat.0, b))
    (matrix2_00[R](Nat.0, a) * matrix2_00[R](Nat.0, b) + matrix2_10[R](Nat.0, a) * matrix2_01[R](Nat.0, b)) +
        (matrix2_01[R](Nat.0, a) * matrix2_10[R](Nat.0, b) + matrix2_11[R](Nat.0, a) * matrix2_11[R](Nat.0, b)) =
        (matrix2_00[R](Nat.0, a) * matrix2_00[R](Nat.0, b) + matrix2_01[R](Nat.0, a) * matrix2_10[R](Nat.0, b)) +
        (matrix2_10[R](Nat.0, a) * matrix2_01[R](Nat.0, b) + matrix2_11[R](Nat.0, a) * matrix2_11[R](Nat.0, b))
    matrix_trace[R](Nat.0.suc.suc, square_matrix_mul[R](Nat.0.suc.suc, b, a)) =
        (matrix2_00[R](Nat.0, a) * matrix2_00[R](Nat.0, b) + matrix2_01[R](Nat.0, a) * matrix2_10[R](Nat.0, b)) +
        (matrix2_10[R](Nat.0, a) * matrix2_01[R](Nat.0, b) + matrix2_11[R](Nat.0, a) * matrix2_11[R](Nat.0, b))
    matrix_trace[R](Nat.0.suc.suc, square_matrix_mul[R](Nat.0.suc.suc, a, b)) =
        matrix_trace[R](Nat.0.suc.suc, square_matrix_mul[R](Nat.0.suc.suc, b, a))
}

/// The entry of a product trace term with the factors swapped.
define trace_mul_comm_fn[R: Ring](
    n: Nat, a: Matrix[R, n.suc, n.suc], b: Matrix[R, n.suc, n.suc], i: Fin[n.suc], k: Fin[n.suc]
) -> R {
    matrix_entry[R](n.suc, n.suc, a, k, i) * matrix_entry[R](n.suc, n.suc, b, i, k)
}

/// A double sum of the swapped product trace terms is invariant under
/// interchanging the summation order.
theorem trace_mul_comm_exchange[R: CommRing](n: Nat, a: Matrix[R, n.suc, n.suc], b: Matrix[R, n.suc, n.suc]) {
    fin_sum[R](n.suc, function(i: Fin[n.suc]) {
        fin_sum[R](n.suc, function(k: Fin[n.suc]) { trace_mul_comm_fn[R](n, a, b, i, k) })
    }) =
        fin_sum[R](n.suc, function(k: Fin[n.suc]) {
            fin_sum[R](n.suc, function(i: Fin[n.suc]) { trace_mul_comm_fn[R](n, a, b, i, k) })
        })
} by {
    fin_sum_suc_eq_sum_fin_enum_suc[R](n, function(i: Fin[n.suc]) {
        fin_sum[R](n.suc, function(k: Fin[n.suc]) { trace_mul_comm_fn[R](n, a, b, i, k) })
    })
    fin_sum[R](n.suc, function(i: Fin[n.suc]) { fin_sum[R](n.suc, function(k: Fin[n.suc]) { trace_mul_comm_fn[R](n, a, b, i, k) }) }) =
        sum[R](map[Fin[n.suc], R](fin_enum_suc(n), function(i: Fin[n.suc]) {
            fin_sum[R](n.suc, function(k: Fin[n.suc]) { trace_mul_comm_fn[R](n, a, b, i, k) })
        }))
    forall(x: Fin[n.suc]) {
        if fin_enum_suc(n).contains(x) {
            fin_sum_suc_eq_sum_fin_enum_suc[R](n, function(k: Fin[n.suc]) { trace_mul_comm_fn[R](n, a, b, x, k) })
            fin_sum[R](n.suc, function(k: Fin[n.suc]) { trace_mul_comm_fn[R](n, a, b, x, k) }) =
                sum[R](map[Fin[n.suc], R](fin_enum_suc(n), function(k: Fin[n.suc]) { trace_mul_comm_fn[R](n, a, b, x, k) }))
        }
    }
    sum_map_of_pointwise[Fin[n.suc], R](fin_enum_suc(n),
        function(i: Fin[n.suc]) { fin_sum[R](n.suc, function(k: Fin[n.suc]) { trace_mul_comm_fn[R](n, a, b, i, k) }) },
        function(i: Fin[n.suc]) { sum[R](map[Fin[n.suc], R](fin_enum_suc(n), function(k: Fin[n.suc]) { trace_mul_comm_fn[R](n, a, b, i, k) })) })
    sum[R](map[Fin[n.suc], R](fin_enum_suc(n), function(i: Fin[n.suc]) {
        fin_sum[R](n.suc, function(k: Fin[n.suc]) { trace_mul_comm_fn[R](n, a, b, i, k) })
    })) =
        sum[R](map[Fin[n.suc], R](fin_enum_suc(n), function(i: Fin[n.suc]) {
            sum[R](map[Fin[n.suc], R](fin_enum_suc(n), function(k: Fin[n.suc]) { trace_mul_comm_fn[R](n, a, b, i, k) }))
        }))
    fin_sum[R](n.suc, function(i: Fin[n.suc]) { fin_sum[R](n.suc, function(k: Fin[n.suc]) { trace_mul_comm_fn[R](n, a, b, i, k) }) }) =
        sum[R](map[Fin[n.suc], R](fin_enum_suc(n), function(i: Fin[n.suc]) {
            sum[R](map[Fin[n.suc], R](fin_enum_suc(n), function(k: Fin[n.suc]) { trace_mul_comm_fn[R](n, a, b, i, k) }))
        }))
    sum_map_sum_exchange[Fin[n.suc], R](fin_enum_suc(n), fin_enum_suc(n), trace_mul_comm_fn[R](n, a, b))
    sum[R](map[Fin[n.suc], R](fin_enum_suc(n), function(i: Fin[n.suc]) {
        sum[R](map[Fin[n.suc], R](fin_enum_suc(n), function(k: Fin[n.suc]) { trace_mul_comm_fn[R](n, a, b, i, k) }))
    })) =
        sum[R](map[Fin[n.suc], R](fin_enum_suc(n), function(k: Fin[n.suc]) {
            sum[R](map[Fin[n.suc], R](fin_enum_suc(n), function(i: Fin[n.suc]) { trace_mul_comm_fn[R](n, a, b, i, k) }))
        }))
    fin_sum_suc_eq_sum_fin_enum_suc[R](n, function(k: Fin[n.suc]) {
        fin_sum[R](n.suc, function(i: Fin[n.suc]) { trace_mul_comm_fn[R](n, a, b, i, k) })
    })
    fin_sum[R](n.suc, function(k: Fin[n.suc]) { fin_sum[R](n.suc, function(i: Fin[n.suc]) { trace_mul_comm_fn[R](n, a, b, i, k) }) }) =
        sum[R](map[Fin[n.suc], R](fin_enum_suc(n), function(k: Fin[n.suc]) {
            fin_sum[R](n.suc, function(i: Fin[n.suc]) { trace_mul_comm_fn[R](n, a, b, i, k) })
        }))
    forall(x: Fin[n.suc]) {
        if fin_enum_suc(n).contains(x) {
            fin_sum_suc_eq_sum_fin_enum_suc[R](n, function(i: Fin[n.suc]) { trace_mul_comm_fn[R](n, a, b, i, x) })
            fin_sum[R](n.suc, function(i: Fin[n.suc]) { trace_mul_comm_fn[R](n, a, b, i, x) }) =
                sum[R](map[Fin[n.suc], R](fin_enum_suc(n), function(i: Fin[n.suc]) { trace_mul_comm_fn[R](n, a, b, i, x) }))
        }
    }
    sum_map_of_pointwise[Fin[n.suc], R](fin_enum_suc(n),
        function(k: Fin[n.suc]) { fin_sum[R](n.suc, function(i: Fin[n.suc]) { trace_mul_comm_fn[R](n, a, b, i, k) }) },
        function(k: Fin[n.suc]) { sum[R](map[Fin[n.suc], R](fin_enum_suc(n), function(i: Fin[n.suc]) { trace_mul_comm_fn[R](n, a, b, i, k) })) })
    sum[R](map[Fin[n.suc], R](fin_enum_suc(n), function(k: Fin[n.suc]) {
        fin_sum[R](n.suc, function(i: Fin[n.suc]) { trace_mul_comm_fn[R](n, a, b, i, k) })
    })) =
        sum[R](map[Fin[n.suc], R](fin_enum_suc(n), function(k: Fin[n.suc]) {
            sum[R](map[Fin[n.suc], R](fin_enum_suc(n), function(i: Fin[n.suc]) { trace_mul_comm_fn[R](n, a, b, i, k) }))
        }))
    sum[R](map[Fin[n.suc], R](fin_enum_suc(n), function(k: Fin[n.suc]) {
        sum[R](map[Fin[n.suc], R](fin_enum_suc(n), function(i: Fin[n.suc]) { trace_mul_comm_fn[R](n, a, b, i, k) }))
    })) =
        fin_sum[R](n.suc, function(k: Fin[n.suc]) { fin_sum[R](n.suc, function(i: Fin[n.suc]) { trace_mul_comm_fn[R](n, a, b, i, k) }) })
    fin_sum[R](n.suc, function(i: Fin[n.suc]) { fin_sum[R](n.suc, function(k: Fin[n.suc]) { trace_mul_comm_fn[R](n, a, b, i, k) }) }) =
        fin_sum[R](n.suc, function(k: Fin[n.suc]) { fin_sum[R](n.suc, function(i: Fin[n.suc]) { trace_mul_comm_fn[R](n, a, b, i, k) }) })
}

/// The trace of a product of square matrices is symmetric in the factors.
theorem matrix_trace_mul_comm[R: CommRing](n: Nat, a: Matrix[R, n.suc, n.suc], b: Matrix[R, n.suc, n.suc]) {
    matrix_trace[R](n.suc, square_matrix_mul[R](n.suc, a, b)) =
        matrix_trace[R](n.suc, square_matrix_mul[R](n.suc, b, a))
} by {
    matrix_trace[R](n.suc, square_matrix_mul[R](n.suc, a, b)) =
        fin_sum[R](n.suc, matrix_trace_term[R](n.suc, square_matrix_mul[R](n.suc, a, b)))
    forall(i: Fin[n.suc]) {
        matrix_entry_mul[R](n.suc, n.suc, n.suc, a, b, i, i)
        matrix_entry[R](n.suc, n.suc, square_matrix_mul[R](n.suc, a, b), i, i) =
            fin_sum[R](n.suc, matrix_mul_term[R](n.suc, n.suc, n.suc, a, b, i, i))
        matrix_trace_term[R](n.suc, square_matrix_mul[R](n.suc, a, b), i) =
            matrix_entry[R](n.suc, n.suc, square_matrix_mul[R](n.suc, a, b), i, i)
        matrix_trace_term[R](n.suc, square_matrix_mul[R](n.suc, a, b), i) =
            fin_sum[R](n.suc, matrix_mul_term[R](n.suc, n.suc, n.suc, a, b, i, i))
    }
    fin_sum_pointwise_eq[R](n.suc, matrix_trace_term[R](n.suc, square_matrix_mul[R](n.suc, a, b)),
        function(i: Fin[n.suc]) { fin_sum[R](n.suc, matrix_mul_term[R](n.suc, n.suc, n.suc, a, b, i, i)) })
    fin_sum[R](n.suc, matrix_trace_term[R](n.suc, square_matrix_mul[R](n.suc, a, b))) =
        fin_sum[R](n.suc, function(i: Fin[n.suc]) { fin_sum[R](n.suc, matrix_mul_term[R](n.suc, n.suc, n.suc, a, b, i, i)) })
    matrix_trace[R](n.suc, square_matrix_mul[R](n.suc, a, b)) =
        fin_sum[R](n.suc, function(i: Fin[n.suc]) { fin_sum[R](n.suc, matrix_mul_term[R](n.suc, n.suc, n.suc, a, b, i, i)) })
    forall(i: Fin[n.suc], k: Fin[n.suc]) {
        matrix_mul_term[R](n.suc, n.suc, n.suc, a, b, i, i, k) =
            matrix_entry[R](n.suc, n.suc, a, i, k) * matrix_entry[R](n.suc, n.suc, b, k, i)
    }
    forall(i: Fin[n.suc]) {
        fin_sum_pointwise_eq[R](n.suc, matrix_mul_term[R](n.suc, n.suc, n.suc, a, b, i, i),
            function(k: Fin[n.suc]) { matrix_entry[R](n.suc, n.suc, a, i, k) * matrix_entry[R](n.suc, n.suc, b, k, i) })
        fin_sum[R](n.suc, matrix_mul_term[R](n.suc, n.suc, n.suc, a, b, i, i)) =
            fin_sum[R](n.suc, function(k: Fin[n.suc]) { matrix_entry[R](n.suc, n.suc, a, i, k) * matrix_entry[R](n.suc, n.suc, b, k, i) })
    }
    forall(i: Fin[n.suc]) {
        fin_sum_pointwise_eq[R](n.suc,
            function(k: Fin[n.suc]) { fin_sum[R](n.suc, matrix_mul_term[R](n.suc, n.suc, n.suc, a, b, k, k)) },
            function(k: Fin[n.suc]) { fin_sum[R](n.suc, function(j: Fin[n.suc]) { matrix_entry[R](n.suc, n.suc, a, k, j) * matrix_entry[R](n.suc, n.suc, b, j, k) }) })
        fin_sum[R](n.suc, function(k: Fin[n.suc]) { fin_sum[R](n.suc, matrix_mul_term[R](n.suc, n.suc, n.suc, a, b, k, k)) }) =
            fin_sum[R](n.suc, function(k: Fin[n.suc]) { fin_sum[R](n.suc, function(j: Fin[n.suc]) { matrix_entry[R](n.suc, n.suc, a, k, j) * matrix_entry[R](n.suc, n.suc, b, j, k) }) })
        matrix_trace[R](n.suc, square_matrix_mul[R](n.suc, a, b)) =
            fin_sum[R](n.suc, function(k: Fin[n.suc]) { fin_sum[R](n.suc, function(j: Fin[n.suc]) { matrix_entry[R](n.suc, n.suc, a, k, j) * matrix_entry[R](n.suc, n.suc, b, j, k) }) })
    }
    matrix_trace[R](n.suc, square_matrix_mul[R](n.suc, a, b)) =
        fin_sum[R](n.suc, function(i: Fin[n.suc]) {
            fin_sum[R](n.suc, function(k: Fin[n.suc]) { matrix_entry[R](n.suc, n.suc, a, i, k) * matrix_entry[R](n.suc, n.suc, b, k, i) })
        })
    matrix_trace[R](n.suc, square_matrix_mul[R](n.suc, b, a)) =
        fin_sum[R](n.suc, matrix_trace_term[R](n.suc, square_matrix_mul[R](n.suc, b, a)))
    forall(i: Fin[n.suc]) {
        matrix_entry_mul[R](n.suc, n.suc, n.suc, b, a, i, i)
        matrix_entry[R](n.suc, n.suc, square_matrix_mul[R](n.suc, b, a), i, i) =
            fin_sum[R](n.suc, matrix_mul_term[R](n.suc, n.suc, n.suc, b, a, i, i))
        matrix_trace_term[R](n.suc, square_matrix_mul[R](n.suc, b, a), i) =
            matrix_entry[R](n.suc, n.suc, square_matrix_mul[R](n.suc, b, a), i, i)
        matrix_trace_term[R](n.suc, square_matrix_mul[R](n.suc, b, a), i) =
            fin_sum[R](n.suc, matrix_mul_term[R](n.suc, n.suc, n.suc, b, a, i, i))
    }
    fin_sum_pointwise_eq[R](n.suc, matrix_trace_term[R](n.suc, square_matrix_mul[R](n.suc, b, a)),
        function(i: Fin[n.suc]) { fin_sum[R](n.suc, matrix_mul_term[R](n.suc, n.suc, n.suc, b, a, i, i)) })
    fin_sum[R](n.suc, matrix_trace_term[R](n.suc, square_matrix_mul[R](n.suc, b, a))) =
        fin_sum[R](n.suc, function(i: Fin[n.suc]) { fin_sum[R](n.suc, matrix_mul_term[R](n.suc, n.suc, n.suc, b, a, i, i)) })
    matrix_trace[R](n.suc, square_matrix_mul[R](n.suc, b, a)) =
        fin_sum[R](n.suc, function(i: Fin[n.suc]) { fin_sum[R](n.suc, matrix_mul_term[R](n.suc, n.suc, n.suc, b, a, i, i)) })
    forall(i: Fin[n.suc], k: Fin[n.suc]) {
        matrix_mul_term[R](n.suc, n.suc, n.suc, b, a, i, i, k) =
            matrix_entry[R](n.suc, n.suc, b, i, k) * matrix_entry[R](n.suc, n.suc, a, k, i)
        matrix_entry[R](n.suc, n.suc, b, i, k) * matrix_entry[R](n.suc, n.suc, a, k, i) =
            matrix_entry[R](n.suc, n.suc, a, k, i) * matrix_entry[R](n.suc, n.suc, b, i, k)
        matrix_mul_term[R](n.suc, n.suc, n.suc, b, a, i, i, k) =
            matrix_entry[R](n.suc, n.suc, a, k, i) * matrix_entry[R](n.suc, n.suc, b, i, k)
    }
    forall(i: Fin[n.suc]) {
        fin_sum_pointwise_eq[R](n.suc, matrix_mul_term[R](n.suc, n.suc, n.suc, b, a, i, i),
            function(k: Fin[n.suc]) { matrix_entry[R](n.suc, n.suc, a, k, i) * matrix_entry[R](n.suc, n.suc, b, i, k) })
        fin_sum[R](n.suc, matrix_mul_term[R](n.suc, n.suc, n.suc, b, a, i, i)) =
            fin_sum[R](n.suc, function(k: Fin[n.suc]) { matrix_entry[R](n.suc, n.suc, a, k, i) * matrix_entry[R](n.suc, n.suc, b, i, k) })
    }
    matrix_trace[R](n.suc, square_matrix_mul[R](n.suc, b, a)) =
        fin_sum[R](n.suc, function(i: Fin[n.suc]) {
            fin_sum[R](n.suc, function(k: Fin[n.suc]) { matrix_entry[R](n.suc, n.suc, a, k, i) * matrix_entry[R](n.suc, n.suc, b, i, k) })
        })
    trace_mul_comm_exchange[R](n, a, b)
    fin_sum[R](n.suc, function(i: Fin[n.suc]) {
        fin_sum[R](n.suc, function(k: Fin[n.suc]) { trace_mul_comm_fn[R](n, a, b, i, k) })
    }) =
        fin_sum[R](n.suc, function(k: Fin[n.suc]) {
            fin_sum[R](n.suc, function(i: Fin[n.suc]) { trace_mul_comm_fn[R](n, a, b, i, k) })
        })
    forall(i: Fin[n.suc], k: Fin[n.suc]) {
        trace_mul_comm_fn[R](n, a, b, i, k) =
            matrix_entry[R](n.suc, n.suc, a, k, i) * matrix_entry[R](n.suc, n.suc, b, i, k)
    }
    forall(i: Fin[n.suc]) {
        fin_sum_pointwise_eq[R](n.suc,
            function(k: Fin[n.suc]) { trace_mul_comm_fn[R](n, a, b, i, k) },
            function(k: Fin[n.suc]) { matrix_entry[R](n.suc, n.suc, a, k, i) * matrix_entry[R](n.suc, n.suc, b, i, k) })
        fin_sum[R](n.suc, function(k: Fin[n.suc]) { trace_mul_comm_fn[R](n, a, b, i, k) }) =
            fin_sum[R](n.suc, function(k: Fin[n.suc]) { matrix_entry[R](n.suc, n.suc, a, k, i) * matrix_entry[R](n.suc, n.suc, b, i, k) })
    }
    fin_sum_pointwise_eq[R](n.suc,
        function(i: Fin[n.suc]) { fin_sum[R](n.suc, function(k: Fin[n.suc]) { trace_mul_comm_fn[R](n, a, b, i, k) }) },
        function(i: Fin[n.suc]) { fin_sum[R](n.suc, function(k: Fin[n.suc]) { matrix_entry[R](n.suc, n.suc, a, k, i) * matrix_entry[R](n.suc, n.suc, b, i, k) }) })
    fin_sum[R](n.suc, function(i: Fin[n.suc]) {
        fin_sum[R](n.suc, function(k: Fin[n.suc]) { trace_mul_comm_fn[R](n, a, b, i, k) })
    }) =
        fin_sum[R](n.suc, function(i: Fin[n.suc]) {
            fin_sum[R](n.suc, function(k: Fin[n.suc]) { matrix_entry[R](n.suc, n.suc, a, k, i) * matrix_entry[R](n.suc, n.suc, b, i, k) })
        })
    forall(k: Fin[n.suc]) {
        fin_sum_pointwise_eq[R](n.suc,
            function(i: Fin[n.suc]) { trace_mul_comm_fn[R](n, a, b, i, k) },
            function(i: Fin[n.suc]) { matrix_entry[R](n.suc, n.suc, a, k, i) * matrix_entry[R](n.suc, n.suc, b, i, k) })
        fin_sum[R](n.suc, function(i: Fin[n.suc]) { trace_mul_comm_fn[R](n, a, b, i, k) }) =
            fin_sum[R](n.suc, function(i: Fin[n.suc]) { matrix_entry[R](n.suc, n.suc, a, k, i) * matrix_entry[R](n.suc, n.suc, b, i, k) })
    }
    fin_sum_pointwise_eq[R](n.suc,
        function(k: Fin[n.suc]) { fin_sum[R](n.suc, function(i: Fin[n.suc]) { trace_mul_comm_fn[R](n, a, b, i, k) }) },
        function(k: Fin[n.suc]) { fin_sum[R](n.suc, function(i: Fin[n.suc]) { matrix_entry[R](n.suc, n.suc, a, k, i) * matrix_entry[R](n.suc, n.suc, b, i, k) }) })
    fin_sum[R](n.suc, function(k: Fin[n.suc]) {
        fin_sum[R](n.suc, function(i: Fin[n.suc]) { trace_mul_comm_fn[R](n, a, b, i, k) })
    }) =
        fin_sum[R](n.suc, function(k: Fin[n.suc]) {
            fin_sum[R](n.suc, function(i: Fin[n.suc]) { matrix_entry[R](n.suc, n.suc, a, k, i) * matrix_entry[R](n.suc, n.suc, b, i, k) })
        })
    fin_sum[R](n.suc, function(k: Fin[n.suc]) {
        fin_sum[R](n.suc, function(i: Fin[n.suc]) { matrix_entry[R](n.suc, n.suc, a, k, i) * matrix_entry[R](n.suc, n.suc, b, i, k) })
    }) =
        fin_sum[R](n.suc, function(i: Fin[n.suc]) {
            fin_sum[R](n.suc, function(k: Fin[n.suc]) { matrix_entry[R](n.suc, n.suc, a, i, k) * matrix_entry[R](n.suc, n.suc, b, k, i) })
        })
    matrix_trace[R](n.suc, square_matrix_mul[R](n.suc, b, a)) =
        fin_sum[R](n.suc, function(i: Fin[n.suc]) {
            fin_sum[R](n.suc, function(k: Fin[n.suc]) { matrix_entry[R](n.suc, n.suc, a, i, k) * matrix_entry[R](n.suc, n.suc, b, k, i) })
        })
    matrix_trace[R](n.suc, square_matrix_mul[R](n.suc, a, b)) =
        matrix_trace[R](n.suc, square_matrix_mul[R](n.suc, b, a))
}
// ---------------------------------------------------------------------------
// Squares of matrix sums and idempotent matrices
// ---------------------------------------------------------------------------

/// The square of a sum of matrices expands into the four products.
theorem square_matrix_mul_add_expand[S: Semiring](n: Nat, a: Matrix[S, n, n], b: Matrix[S, n, n]) {
    square_matrix_mul[S](n, matrix_add[S](n, n, a, b), matrix_add[S](n, n, a, b)) =
        square_matrix_mul[S](n, a, a) + square_matrix_mul[S](n, b, a) +
            square_matrix_mul[S](n, a, b) + square_matrix_mul[S](n, b, b)
} by {
    square_matrix_mul_add_right[S](n, matrix_add[S](n, n, a, b), a, b)
    square_matrix_mul[S](n, matrix_add[S](n, n, a, b), matrix_add[S](n, n, a, b)) =
        square_matrix_mul[S](n, matrix_add[S](n, n, a, b), a) +
        square_matrix_mul[S](n, matrix_add[S](n, n, a, b), b)
    square_matrix_mul_add_left[S](n, a, b, a)
    square_matrix_mul[S](n, matrix_add[S](n, n, a, b), a) =
        square_matrix_mul[S](n, a, a) + square_matrix_mul[S](n, b, a)
    square_matrix_mul_add_left[S](n, a, b, b)
    square_matrix_mul[S](n, matrix_add[S](n, n, a, b), b) =
        square_matrix_mul[S](n, a, b) + square_matrix_mul[S](n, b, b)
    square_matrix_mul[S](n, matrix_add[S](n, n, a, b), matrix_add[S](n, n, a, b)) =
        (square_matrix_mul[S](n, a, a) + square_matrix_mul[S](n, b, a)) +
        (square_matrix_mul[S](n, a, b) + square_matrix_mul[S](n, b, b))
    square_matrix_mul[S](n, matrix_add[S](n, n, a, b), matrix_add[S](n, n, a, b)) =
        square_matrix_mul[S](n, a, a) + square_matrix_mul[S](n, b, a) +
            square_matrix_mul[S](n, a, b) + square_matrix_mul[S](n, b, b)
}

/// If two matrices commute, the square of their sum collects the cross terms.
theorem square_matrix_mul_comm_add_expand[R: CommRing](n: Nat, a: Matrix[R, n, n], b: Matrix[R, n, n]) {
    square_matrix_mul[R](n, a, b) = square_matrix_mul[R](n, b, a) implies
        square_matrix_mul[R](n, matrix_add[R](n, n, a, b), matrix_add[R](n, n, a, b)) =
            square_matrix_mul[R](n, a, a) +
            matrix_add[R](n, n, square_matrix_mul[R](n, a, b), square_matrix_mul[R](n, a, b)) +
            square_matrix_mul[R](n, b, b)
} by {
    if square_matrix_mul[R](n, a, b) = square_matrix_mul[R](n, b, a) {
        square_matrix_mul_add_expand[R](n, a, b)
        square_matrix_mul[R](n, matrix_add[R](n, n, a, b), matrix_add[R](n, n, a, b)) =
            square_matrix_mul[R](n, a, a) + square_matrix_mul[R](n, b, a) +
                square_matrix_mul[R](n, a, b) + square_matrix_mul[R](n, b, b)
        square_matrix_mul[R](n, a, a) + square_matrix_mul[R](n, b, a) +
            square_matrix_mul[R](n, a, b) + square_matrix_mul[R](n, b, b) =
            square_matrix_mul[R](n, a, a) +
            matrix_add[R](n, n, square_matrix_mul[R](n, a, b), square_matrix_mul[R](n, a, b)) +
            square_matrix_mul[R](n, b, b)
        square_matrix_mul[R](n, matrix_add[R](n, n, a, b), matrix_add[R](n, n, a, b)) =
            square_matrix_mul[R](n, a, a) +
            matrix_add[R](n, n, square_matrix_mul[R](n, a, b), square_matrix_mul[R](n, a, b)) +
            square_matrix_mul[R](n, b, b)
    }
}

/// The eigenvalue of an idempotent matrix is zero or one: if `A^2 = A` and
/// `v` is an eigenvector of `A` for `lam`, then `lam = 0` or `lam = 1`.
theorem idempotent_eigenvalue_zero_or_one[F: Field](
    a: Matrix[F, Nat.0.suc.suc, Nat.0.suc.suc], lam: F, v: Fin[Nat.0.suc.suc] -> F
) {
    square_matrix_mul[F](Nat.0.suc.suc, a, a) = a and
    is_eigenvector[F](Nat.0.suc.suc, a, lam, v)
        implies lam = F.0 or lam = F.1
} by {
    if square_matrix_mul[F](Nat.0.suc.suc, a, a) = a and
        is_eigenvector[F](Nat.0.suc.suc, a, lam, v) {
        is_eigenvector_nonzero[F](Nat.0.suc.suc, a, lam, v)
        is_nonzero_vector_witness[F](Nat.0.suc.suc, v)
        let (j: Fin[Nat.0.suc.suc]) satisfy { v(j) != F.0 }
        matrix_apply_mul_two[F](a, a, v, j)
        matrix_apply[F](Nat.0.suc.suc, square_matrix_mul[F](Nat.0.suc.suc, a, a), v, j) =
            matrix_apply[F](Nat.0.suc.suc, a, matrix_apply_vec[F](Nat.0.suc.suc, a, v), j)
        matrix_apply[F](Nat.0.suc.suc, a, v, j) =
            matrix_apply[F](Nat.0.suc.suc, square_matrix_mul[F](Nat.0.suc.suc, a, a), v, j)
        matrix_apply[F](Nat.0.suc.suc, a, v, j) =
            matrix_apply[F](Nat.0.suc.suc, a, matrix_apply_vec[F](Nat.0.suc.suc, a, v), j)
        is_eigenvector_apply[F](Nat.0.suc.suc, a, lam, v, j)
        matrix_apply[F](Nat.0.suc.suc, a, v, j) = lam * v(j)
        forall(k: Fin[Nat.0.suc.suc]) {
            matrix_apply_vec[F](Nat.0.suc.suc, a, v, k) = matrix_apply[F](Nat.0.suc.suc, a, v, k)
            is_eigenvector_apply[F](Nat.0.suc.suc, a, lam, v, k)
            matrix_apply[F](Nat.0.suc.suc, a, v, k) = lam * v(k)
            matrix_apply_vec[F](Nat.0.suc.suc, a, v, k) = lam * v(k)
            fin_vector_smul_apply[F, F](ring_as_module[F], Nat.0.suc.suc, lam, v, k)
            fin_vector_smul[F, F](ring_as_module[F], Nat.0.suc.suc, lam, v, k) = lam * v(k)
            matrix_apply_vec[F](Nat.0.suc.suc, a, v, k) =
                fin_vector_smul[F, F](ring_as_module[F], Nat.0.suc.suc, lam, v, k)
        }
        matrix_apply_pointwise_eq[F](Nat.0.suc.suc, a, matrix_apply_vec[F](Nat.0.suc.suc, a, v),
            fin_vector_smul[F, F](ring_as_module[F], Nat.0.suc.suc, lam, v), j)
        matrix_apply[F](Nat.0.suc.suc, a, matrix_apply_vec[F](Nat.0.suc.suc, a, v), j) =
            matrix_apply[F](Nat.0.suc.suc, a, fin_vector_smul[F, F](ring_as_module[F], Nat.0.suc.suc, lam, v), j)
        matrix_apply_smul[F](Nat.0.suc.suc, a, lam, v, j)
        matrix_apply[F](Nat.0.suc.suc, a, fin_vector_smul[F, F](ring_as_module[F], Nat.0.suc.suc, lam, v), j) =
            lam * matrix_apply[F](Nat.0.suc.suc, a, v, j)
        matrix_apply[F](Nat.0.suc.suc, a, matrix_apply_vec[F](Nat.0.suc.suc, a, v), j) =
            lam * matrix_apply[F](Nat.0.suc.suc, a, v, j)
        matrix_apply[F](Nat.0.suc.suc, a, v, j) =
            lam * matrix_apply[F](Nat.0.suc.suc, a, v, j)
        matrix_apply[F](Nat.0.suc.suc, a, v, j) = lam * v(j)
        lam * matrix_apply[F](Nat.0.suc.suc, a, v, j) = lam * (lam * v(j))
        matrix_apply[F](Nat.0.suc.suc, a, v, j) = lam * (lam * v(j))
        matrix_apply[F](Nat.0.suc.suc, a, v, j) = (lam * lam) * v(j)
        lam * v(j) = (lam * lam) * v(j)
        mul_sub_right[F](lam * lam, lam, v(j))
        (lam * lam - lam) * v(j) = (lam * lam) * v(j) - lam * v(j)
        (lam * lam) * v(j) - lam * v(j) = F.0
        (lam * lam - lam) * v(j) = F.0
        field_mul_eq_zero[F](lam * lam - lam, v(j))
        lam * lam - lam = F.0 or v(j) = F.0
        if lam * lam - lam = F.0 {
            lam * lam - lam = F.0
        } else {
            v(j) = F.0
            false
        }
        lam * lam - lam = F.0
        sub_factor[F](lam, lam, F.1)
        lam * lam - lam * F.1 = lam * (lam - F.1)
        lam * F.1 = lam
        lam * lam - lam = lam * (lam - F.1)
        lam * (lam - F.1) = F.0
        field_mul_eq_zero[F](lam, lam - F.1)
        lam = F.0 or lam - F.1 = F.0
        if lam = F.0 {
            lam = F.0 or lam = F.1
        } else {
            lam - F.1 = F.0
            ring_sub_eq_zero_iff_eq[F](lam, F.1)
            lam = F.1
            lam = F.0 or lam = F.1
        }
        lam = F.0 or lam = F.1
    }
}
