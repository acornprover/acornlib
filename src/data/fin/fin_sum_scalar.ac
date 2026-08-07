from nat import Nat
from data.fin.fin import Fin, fin_new_exists_of_lt, new_round_trip
from list import partial, partial_zero, partial_split_last, partial_pointwise_eq
from semiring import Semiring
from data.fin.fin_sum import fin_sum, fin_sum_apply, fin_value_or_zero, fin_value_or_zero_value

/// A sequence scaled termwise.
define scale_seq[S: Semiring](c: S, f: Nat -> S) -> (Nat -> S) {
    function(k: Nat) {
        c * f(k)
    }
}

/// A scalar factors out of a partial sum.
theorem partial_scalar_mul[S: Semiring](c: S, f: Nat -> S, n: Nat) {
    partial(scale_seq[S](c, f), n) = c * partial(f, n)
} by {
    define p(x: Nat) -> Bool {
        partial(scale_seq[S](c, f), x) = c * partial(f, x)
    }
    partial_zero(scale_seq[S](c, f))
    partial(scale_seq[S](c, f), Nat.0) = S.0
    partial_zero(f)
    partial(f, Nat.0) = S.0
    (c * S.0 = S.0)
    p(Nat.0)
    forall(m: Nat) {
        if p(m) {
            partial(scale_seq[S](c, f), m) = c * partial(f, m)
            partial_split_last(scale_seq[S](c, f), m)
            (partial(scale_seq[S](c, f), m.suc)
                = partial(scale_seq[S](c, f), m) + scale_seq[S](c, f)(m))
            (scale_seq[S](c, f)(m) = c * f(m))
            (partial(scale_seq[S](c, f), m.suc) = c * partial(f, m) + c * f(m))
            (c * partial(f, m) + c * f(m) = c * (partial(f, m) + f(m)))
            partial_split_last(f, m)
            partial(f, m.suc) = partial(f, m) + f(m)
            partial(scale_seq[S](c, f), m.suc) = c * partial(f, m.suc)
            p(m.suc)
        }
        (p(m) implies p(m.suc))
    }
    p(Nat.0) and forall(m: Nat) {
        p(m) implies p(m.suc)
    }
    Nat.induction(p)
    p(n)
}

/// A function on `Fin[n]` scaled termwise.
define fin_scale[S: Semiring](n: Nat, c: S, f: Fin[n] -> S) -> (Fin[n] -> S) {
    function(i: Fin[n]) {
        c * f(i)
    }
}

/// Zero extension commutes with scaling, below the bound.
theorem fin_value_or_zero_scale[S: Semiring](n: Nat, c: S, f: Fin[n] -> S, k: Nat) {
    k < n implies fin_value_or_zero[S](n, fin_scale[S](n, c, f), k)
        = c * fin_value_or_zero[S](n, f, k)
} by {
    if k < n {
        fin_new_exists_of_lt(n, k)
        exists(j: Fin[n]) {
            Fin[n].new(k) = Option.some(j)
        }
        let (i: Fin[n]) satisfy {
            Fin[n].new(k) = Option.some(i)
        }
        new_round_trip(n, k, i)
        i.value = k
        fin_value_or_zero_value[S](n, fin_scale[S](n, c, f), i)
        (fin_value_or_zero[S](n, fin_scale[S](n, c, f), i.value)
            = fin_scale[S](n, c, f)(i))
        (fin_value_or_zero[S](n, fin_scale[S](n, c, f), k) = fin_scale[S](n, c, f)(i))
        (fin_scale[S](n, c, f)(i) = c * f(i))
        fin_value_or_zero_value[S](n, f, i)
        fin_value_or_zero[S](n, f, i.value) = f(i)
        fin_value_or_zero[S](n, f, k) = f(i)
        (fin_value_or_zero[S](n, fin_scale[S](n, c, f), k)
            = c * fin_value_or_zero[S](n, f, k))
    }
}

/// A scalar factors out of a finite sum.
///
/// The distributive law over a finite index set. `src/nat_range_sum_semiring.ac` has this for
/// range sums; this is the `Fin`-indexed form, which is what a matrix row needs.
theorem fin_sum_scalar_mul[S: Semiring](n: Nat, c: S, f: Fin[n] -> S) {
    fin_sum[S](n, fin_scale[S](n, c, f)) = c * fin_sum[S](n, f)
} by {
    fin_sum_apply[S](n, fin_scale[S](n, c, f))
    (fin_sum[S](n, fin_scale[S](n, c, f))
        = partial(fin_value_or_zero[S](n, fin_scale[S](n, c, f)), n))
    forall(k: Nat) {
        if k < n {
            fin_value_or_zero_scale[S](n, c, f, k)
            (fin_value_or_zero[S](n, fin_scale[S](n, c, f), k)
                = c * fin_value_or_zero[S](n, f, k))
            (scale_seq[S](c, fin_value_or_zero[S](n, f))(k)
                = c * fin_value_or_zero[S](n, f, k))
            (fin_value_or_zero[S](n, fin_scale[S](n, c, f), k)
                = scale_seq[S](c, fin_value_or_zero[S](n, f))(k))
        }
        (k < n implies fin_value_or_zero[S](n, fin_scale[S](n, c, f), k)
            = scale_seq[S](c, fin_value_or_zero[S](n, f))(k))
    }
    partial_pointwise_eq[S](fin_value_or_zero[S](n, fin_scale[S](n, c, f)),
        scale_seq[S](c, fin_value_or_zero[S](n, f)), n)
    (partial(fin_value_or_zero[S](n, fin_scale[S](n, c, f)), n)
        = partial(scale_seq[S](c, fin_value_or_zero[S](n, f)), n))
    partial_scalar_mul[S](c, fin_value_or_zero[S](n, f), n)
    (partial(scale_seq[S](c, fin_value_or_zero[S](n, f)), n)
        = c * partial(fin_value_or_zero[S](n, f), n))
    fin_sum_apply[S](n, f)
    fin_sum[S](n, f) = partial(fin_value_or_zero[S](n, f), n)
    fin_sum[S](n, fin_scale[S](n, c, f)) = c * fin_sum[S](n, f)
}
