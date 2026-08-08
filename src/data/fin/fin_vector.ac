from nat import Nat
from data.fin.fin import Fin
from algebra.add import Add
from algebra.zero import Zero
from algebra.neg import Neg
from algebra.add_semigroup import AddSemigroup
from algebra.add_comm_semigroup import AddCommSemigroup
from algebra.add_monoid import AddMonoid
from algebra.add_group import AddGroup
from algebra.add_comm_group import AddCommGroup
from algebra.ring.ring import Ring
from algebra.module.module import Module, module_smul_add_left, module_smul_add_right, module_smul_assoc,
    module_smul_one, module_smul_zero_left, module_smul_zero_right, module_smul_neg_left,
    module_smul_neg_right
from data.basic.functions import function_extensionality

/// The zero coordinate vector of length `n`.
define fin_vector_zero[A: Zero](n: Nat, i: Fin[n]) -> A {
    A.0
}

/// Pointwise addition of coordinate vectors of length `n`.
define fin_vector_add[A: Add](n: Nat, v: Fin[n] -> A, w: Fin[n] -> A, i: Fin[n]) -> A {
    v(i) + w(i)
}

/// Pointwise negation of a coordinate vector of length `n`.
define fin_vector_neg[A: Neg](n: Nat, v: Fin[n] -> A, i: Fin[n]) -> A {
    -v(i)
}

/// Pointwise subtraction of coordinate vectors of length `n`.
define fin_vector_sub[A: AddGroup](n: Nat, v: Fin[n] -> A, w: Fin[n] -> A, i: Fin[n]) -> A {
    v(i) - w(i)
}

/// Scalar multiplication of a coordinate vector in a module.
define fin_vector_smul[R: Ring, M: AddCommGroup](
    carrier: Module[R, M], n: Nat, r: R, v: Fin[n] -> M, i: Fin[n]
) -> M {
    carrier.smul(r, v(i))
}

/// Coordinate-vector extensionality.
theorem fin_vector_ext[A](n: Nat, v: Fin[n] -> A, w: Fin[n] -> A) {
    (forall(i: Fin[n]) { v(i) = w(i) }) implies v = w
} by {
    function_extensionality(v, w)
}

/// The zero coordinate vector has zero in every coordinate.
theorem fin_vector_zero_apply[A: Zero](n: Nat, i: Fin[n]) {
    fin_vector_zero[A](n, i) = A.0
}

/// Equal coordinate vectors have equal values in every coordinate.
theorem fin_vector_eq_apply[A](n: Nat, v: Fin[n] -> A, w: Fin[n] -> A, i: Fin[n]) {
    v = w implies v(i) = w(i)
}

/// Pointwise addition agrees with addition of coordinates.
theorem fin_vector_add_apply[A: Add](n: Nat, v: Fin[n] -> A, w: Fin[n] -> A, i: Fin[n]) {
    fin_vector_add[A](n, v, w, i) = v(i) + w(i)
}

/// Pointwise negation agrees with negation of coordinates.
theorem fin_vector_neg_apply[A: Neg](n: Nat, v: Fin[n] -> A, i: Fin[n]) {
    fin_vector_neg[A](n, v, i) = -v(i)
}

/// Pointwise subtraction agrees with subtraction of coordinates.
theorem fin_vector_sub_apply[A: AddGroup](n: Nat, v: Fin[n] -> A, w: Fin[n] -> A, i: Fin[n]) {
    fin_vector_sub[A](n, v, w, i) = v(i) - w(i)
}

/// Scalar multiplication agrees with scalar multiplication of coordinates.
theorem fin_vector_smul_apply[R: Ring, M: AddCommGroup](
    carrier: Module[R, M], n: Nat, r: R, v: Fin[n] -> M, i: Fin[n]
) {
    fin_vector_smul[R, M](carrier, n, r, v, i) = carrier.smul(r, v(i))
}

/// Pointwise addition of coordinate vectors is associative at each coordinate.
theorem fin_vector_add_assoc_at[A: AddSemigroup](
    n: Nat, u: Fin[n] -> A, v: Fin[n] -> A, w: Fin[n] -> A, i: Fin[n]
) {
    fin_vector_add[A](n, u, fin_vector_add[A](n, v, w), i) =
        fin_vector_add[A](n, fin_vector_add[A](n, u, v), w, i)
} by {
    u(i) + (v(i) + w(i)) = (u(i) + v(i)) + w(i)
}

/// Pointwise addition of coordinate vectors is commutative at each coordinate.
theorem fin_vector_add_comm_at[A: AddCommSemigroup](n: Nat, v: Fin[n] -> A, w: Fin[n] -> A, i: Fin[n]) {
    fin_vector_add[A](n, v, w, i) = fin_vector_add[A](n, w, v, i)
} by {
    v(i) + w(i) = w(i) + v(i)
}

/// Adding the zero coordinate vector on the right changes nothing at each coordinate.
theorem fin_vector_add_zero_right_at[A: AddMonoid](n: Nat, v: Fin[n] -> A, i: Fin[n]) {
    fin_vector_add[A](n, v, fin_vector_zero[A](n), i) = v(i)
} by {
    fin_vector_add[A](n, v, fin_vector_zero[A](n), i) = v(i) + fin_vector_zero[A](n, i)
    fin_vector_zero[A](n, i) = A.0
    v(i) + A.0 = v(i)
}

/// Adding the zero coordinate vector on the left changes nothing at each coordinate.
theorem fin_vector_add_zero_left_at[A: AddMonoid](n: Nat, v: Fin[n] -> A, i: Fin[n]) {
    fin_vector_add[A](n, fin_vector_zero[A](n), v, i) = v(i)
} by {
    fin_vector_add[A](n, fin_vector_zero[A](n), v, i) = fin_vector_zero[A](n, i) + v(i)
    fin_vector_zero[A](n, i) = A.0
    A.0 + v(i) = v(i)
}

/// Pointwise negation is a right additive inverse at each coordinate.
theorem fin_vector_add_neg_right_at[A: AddGroup](n: Nat, v: Fin[n] -> A, i: Fin[n]) {
    fin_vector_add[A](n, v, fin_vector_neg[A](n, v), i) = fin_vector_zero[A](n, i)
} by {
    fin_vector_add[A](n, v, fin_vector_neg[A](n, v), i) = v(i) + fin_vector_neg[A](n, v, i)
    fin_vector_neg[A](n, v, i) = -v(i)
    v(i) + -v(i) = A.0
    fin_vector_zero[A](n, i) = A.0
}

/// Pointwise negation is a left additive inverse at each coordinate.
theorem fin_vector_add_neg_left_at[A: AddGroup](n: Nat, v: Fin[n] -> A, i: Fin[n]) {
    fin_vector_add[A](n, fin_vector_neg[A](n, v), v, i) = fin_vector_zero[A](n, i)
} by {
    fin_vector_add[A](n, fin_vector_neg[A](n, v), v, i) = fin_vector_neg[A](n, v, i) + v(i)
    fin_vector_neg[A](n, v, i) = -v(i)
    -v(i) + v(i) = A.0
    fin_vector_zero[A](n, i) = A.0
}

/// Pointwise subtraction is pointwise addition with pointwise negation at each coordinate.
theorem fin_vector_sub_eq_add_neg_at[A: AddGroup](n: Nat, v: Fin[n] -> A, w: Fin[n] -> A, i: Fin[n]) {
    fin_vector_sub[A](n, v, w, i) = fin_vector_add[A](n, v, fin_vector_neg[A](n, w), i)
} by {
    fin_vector_sub[A](n, v, w, i) = v(i) - w(i)
    v(i) - w(i) = v(i) + -w(i)
    fin_vector_neg[A](n, w, i) = -w(i)
    fin_vector_add[A](n, v, fin_vector_neg[A](n, w), i) = v(i) + fin_vector_neg[A](n, w, i)
}

/// The identity scalar fixes every coordinate.
theorem fin_vector_smul_one_at[R: Ring, M: AddCommGroup](carrier: Module[R, M], n: Nat, v: Fin[n] -> M, i: Fin[n]) {
    fin_vector_smul[R, M](carrier, n, R.1, v, i) = v(i)
} by {
    fin_vector_smul[R, M](carrier, n, R.1, v, i) = carrier.smul(R.1, v(i))
    module_smul_one(carrier, v(i))
}

/// The zero scalar annihilates every coordinate.
theorem fin_vector_smul_zero_left_at[R: Ring, M: AddCommGroup](carrier: Module[R, M], n: Nat, v: Fin[n] -> M, i: Fin[n]) {
    fin_vector_smul[R, M](carrier, n, R.0, v, i) = M.0
} by {
    fin_vector_smul[R, M](carrier, n, R.0, v, i) = carrier.smul(R.0, v(i))
    module_smul_zero_left(carrier, v(i))
    carrier.smul(R.0, v(i)) = M.0
}

/// Every scalar annihilates a zero coordinate.
theorem fin_vector_smul_zero_right_coord[R: Ring, M: AddCommGroup](carrier: Module[R, M], n: Nat, r: R, i: Fin[n]) {
    carrier.smul(r, M.0) = M.0
} by {
    module_smul_zero_right(carrier, r)
    carrier.smul(r, M.0) = M.0
}

/// Scalar multiplication distributes over pointwise negation at each coordinate.
theorem fin_vector_smul_neg_coord[R: Ring, M: AddCommGroup](carrier: Module[R, M], n: Nat, r: R, v: Fin[n] -> M, i: Fin[n]) {
    carrier.smul(r, -v(i)) = -fin_vector_smul[R, M](carrier, n, r, v, i)
} by {
    module_smul_neg_right(carrier, r, v(i))
    carrier.smul(r, -v(i)) = -carrier.smul(r, v(i))
    fin_vector_smul[R, M](carrier, n, r, v, i) = carrier.smul(r, v(i))
}

/// Negating the scalar negates scalar multiplication at each coordinate.
theorem fin_vector_smul_neg_left_at[R: Ring, M: AddCommGroup](carrier: Module[R, M], n: Nat, r: R, v: Fin[n] -> M, i: Fin[n]) {
    fin_vector_smul[R, M](carrier, n, -r, v, i) =
        fin_vector_neg[M](n, fin_vector_smul[R, M](carrier, n, r, v), i)
} by {
    fin_vector_smul[R, M](carrier, n, -r, v, i) = carrier.smul(-r, v(i))
    module_smul_neg_left(carrier, r, v(i))
    carrier.smul(-r, v(i)) = -carrier.smul(r, v(i))
    fin_vector_smul[R, M](carrier, n, r, v, i) = carrier.smul(r, v(i))
    fin_vector_neg[M](n, fin_vector_smul[R, M](carrier, n, r, v), i) =
        -fin_vector_smul[R, M](carrier, n, r, v, i)
}

/// Scalar multiplication distributes over scalar addition at each coordinate.
theorem fin_vector_smul_add_left_at[R: Ring, M: AddCommGroup](
    carrier: Module[R, M], n: Nat, r: R, s: R, v: Fin[n] -> M, i: Fin[n]
) {
    fin_vector_smul[R, M](carrier, n, r + s, v, i) =
        fin_vector_add[M](n, fin_vector_smul[R, M](carrier, n, r, v), fin_vector_smul[R, M](carrier, n, s, v), i)
} by {
    fin_vector_smul[R, M](carrier, n, r + s, v, i) = carrier.smul(r + s, v(i))
    module_smul_add_left(carrier, r, s, v(i))
    carrier.smul(r + s, v(i)) = carrier.smul(r, v(i)) + carrier.smul(s, v(i))
    fin_vector_add[M](n, fin_vector_smul[R, M](carrier, n, r, v), fin_vector_smul[R, M](carrier, n, s, v), i) =
        fin_vector_smul[R, M](carrier, n, r, v, i) + fin_vector_smul[R, M](carrier, n, s, v, i)
    fin_vector_smul[R, M](carrier, n, r, v, i) = carrier.smul(r, v(i))
    fin_vector_smul[R, M](carrier, n, s, v, i) = carrier.smul(s, v(i))
}

/// Scalar multiplication distributes over coordinate-vector addition at each coordinate.
theorem fin_vector_smul_add_right_at[R: Ring, M: AddCommGroup](
    carrier: Module[R, M], n: Nat, r: R, v: Fin[n] -> M, w: Fin[n] -> M, i: Fin[n]
) {
    fin_vector_smul[R, M](carrier, n, r, fin_vector_add[M](n, v, w), i) =
        fin_vector_add[M](n, fin_vector_smul[R, M](carrier, n, r, v), fin_vector_smul[R, M](carrier, n, r, w), i)
} by {
    fin_vector_smul[R, M](carrier, n, r, fin_vector_add[M](n, v, w), i) =
        carrier.smul(r, fin_vector_add[M](n, v, w, i))
    fin_vector_add[M](n, v, w, i) = v(i) + w(i)
    module_smul_add_right(carrier, r, v(i), w(i))
    carrier.smul(r, v(i) + w(i)) = carrier.smul(r, v(i)) + carrier.smul(r, w(i))
    fin_vector_add[M](n, fin_vector_smul[R, M](carrier, n, r, v), fin_vector_smul[R, M](carrier, n, r, w), i) =
        fin_vector_smul[R, M](carrier, n, r, v, i) + fin_vector_smul[R, M](carrier, n, r, w, i)
    fin_vector_smul[R, M](carrier, n, r, v, i) = carrier.smul(r, v(i))
    fin_vector_smul[R, M](carrier, n, r, w, i) = carrier.smul(r, w(i))
}

/// Scalar multiplication is compatible with scalar multiplication at each coordinate.
theorem fin_vector_smul_assoc_at[R: Ring, M: AddCommGroup](
    carrier: Module[R, M], n: Nat, r: R, s: R, v: Fin[n] -> M, i: Fin[n]
) {
    fin_vector_smul[R, M](carrier, n, r * s, v, i) =
        fin_vector_smul[R, M](carrier, n, r, fin_vector_smul[R, M](carrier, n, s, v), i)
} by {
    fin_vector_smul[R, M](carrier, n, r * s, v, i) = carrier.smul(r * s, v(i))
    module_smul_assoc(carrier, r, s, v(i))
    carrier.smul(r * s, v(i)) = carrier.smul(r, carrier.smul(s, v(i)))
    fin_vector_smul[R, M](carrier, n, s, v, i) = carrier.smul(s, v(i))
    fin_vector_smul[R, M](carrier, n, r, fin_vector_smul[R, M](carrier, n, s, v), i) =
        carrier.smul(r, fin_vector_smul[R, M](carrier, n, s, v, i))
}

/// Every scalar annihilates the zero coordinate vector at each coordinate.
theorem fin_vector_smul_zero_right_at[R: Ring, M: AddCommGroup](carrier: Module[R, M], n: Nat, r: R, i: Fin[n]) {
    fin_vector_smul[R, M](carrier, n, r, fin_vector_zero[M](n), i) = fin_vector_zero[M](n, i)
} by {
    fin_vector_smul[R, M](carrier, n, r, fin_vector_zero[M](n), i) =
        carrier.smul(r, fin_vector_zero[M](n, i))
    fin_vector_zero[M](n, i) = M.0
    fin_vector_smul_zero_right_coord[R, M](carrier, n, r, i)
}

/// Scalar multiplication distributes over pointwise negation in fin-vector form at each coordinate.
theorem fin_vector_smul_neg_right_at[R: Ring, M: AddCommGroup](carrier: Module[R, M], n: Nat, r: R, v: Fin[n] -> M, i: Fin[n]) {
    fin_vector_smul[R, M](carrier, n, r, fin_vector_neg[M](n, v), i) =
        fin_vector_neg[M](n, fin_vector_smul[R, M](carrier, n, r, v), i)
} by {
    fin_vector_smul[R, M](carrier, n, r, fin_vector_neg[M](n, v), i) =
        carrier.smul(r, fin_vector_neg[M](n, v, i))
    fin_vector_neg[M](n, v, i) = -v(i)
    fin_vector_smul_neg_coord[R, M](carrier, n, r, v, i)
    fin_vector_neg[M](n, fin_vector_smul[R, M](carrier, n, r, v), i) =
        -fin_vector_smul[R, M](carrier, n, r, v, i)
}

/// Pointwise addition of coordinate vectors is associative.
theorem fin_vector_add_assoc[A: AddSemigroup](n: Nat, u: Fin[n] -> A, v: Fin[n] -> A, w: Fin[n] -> A) {
    fin_vector_add[A](n, u, fin_vector_add[A](n, v, w)) =
        fin_vector_add[A](n, fin_vector_add[A](n, u, v), w)
} by {
    let lhs = fin_vector_add[A](n, u, fin_vector_add[A](n, v, w))
    let rhs = fin_vector_add[A](n, fin_vector_add[A](n, u, v), w)
    forall(i: Fin[n]) {
        fin_vector_add_assoc_at[A](n, u, v, w, i)
        lhs(i) = rhs(i)
    }
    fin_vector_ext[A](n, lhs, rhs)
}

/// Pointwise addition of coordinate vectors is commutative.
theorem fin_vector_add_comm[A: AddCommSemigroup](n: Nat, v: Fin[n] -> A, w: Fin[n] -> A) {
    fin_vector_add[A](n, v, w) = fin_vector_add[A](n, w, v)
} by {
    let lhs = fin_vector_add[A](n, v, w)
    let rhs = fin_vector_add[A](n, w, v)
    forall(i: Fin[n]) {
        fin_vector_add_comm_at[A](n, v, w, i)
        lhs(i) = rhs(i)
    }
    fin_vector_ext[A](n, lhs, rhs)
}

/// Adding the zero coordinate vector on the right changes nothing.
theorem fin_vector_add_zero_right[A: AddMonoid](n: Nat, v: Fin[n] -> A) {
    fin_vector_add[A](n, v, fin_vector_zero[A](n)) = v
} by {
    let lhs = fin_vector_add[A](n, v, fin_vector_zero[A](n))
    forall(i: Fin[n]) {
        fin_vector_add_zero_right_at[A](n, v, i)
        lhs(i) = v(i)
    }
    fin_vector_ext[A](n, lhs, v)
}

/// Adding the zero coordinate vector on the left changes nothing.
theorem fin_vector_add_zero_left[A: AddMonoid](n: Nat, v: Fin[n] -> A) {
    fin_vector_add[A](n, fin_vector_zero[A](n), v) = v
} by {
    let lhs = fin_vector_add[A](n, fin_vector_zero[A](n), v)
    forall(i: Fin[n]) {
        fin_vector_add_zero_left_at[A](n, v, i)
        lhs(i) = v(i)
    }
    fin_vector_ext[A](n, lhs, v)
}

/// Pointwise negation is a right additive inverse.
theorem fin_vector_add_neg_right[A: AddGroup](n: Nat, v: Fin[n] -> A) {
    fin_vector_add[A](n, v, fin_vector_neg[A](n, v)) = fin_vector_zero[A](n)
} by {
    let lhs = fin_vector_add[A](n, v, fin_vector_neg[A](n, v))
    let rhs = fin_vector_zero[A](n)
    forall(i: Fin[n]) {
        fin_vector_add_neg_right_at[A](n, v, i)
        lhs(i) = rhs(i)
    }
    fin_vector_ext[A](n, lhs, rhs)
}

/// Pointwise negation is a left additive inverse.
theorem fin_vector_add_neg_left[A: AddGroup](n: Nat, v: Fin[n] -> A) {
    fin_vector_add[A](n, fin_vector_neg[A](n, v), v) = fin_vector_zero[A](n)
} by {
    let lhs = fin_vector_add[A](n, fin_vector_neg[A](n, v), v)
    let rhs = fin_vector_zero[A](n)
    forall(i: Fin[n]) {
        fin_vector_add_neg_left_at[A](n, v, i)
        lhs(i) = rhs(i)
    }
    fin_vector_ext[A](n, lhs, rhs)
}

/// Pointwise subtraction is pointwise addition with pointwise negation.
theorem fin_vector_sub_eq_add_neg[A: AddGroup](n: Nat, v: Fin[n] -> A, w: Fin[n] -> A) {
    fin_vector_sub[A](n, v, w) = fin_vector_add[A](n, v, fin_vector_neg[A](n, w))
} by {
    let lhs = fin_vector_sub[A](n, v, w)
    let rhs = fin_vector_add[A](n, v, fin_vector_neg[A](n, w))
    forall(i: Fin[n]) {
        fin_vector_sub_eq_add_neg_at[A](n, v, w, i)
        lhs(i) = rhs(i)
    }
    fin_vector_ext[A](n, lhs, rhs)
}

/// The identity scalar fixes a coordinate vector.
theorem fin_vector_smul_one[R: Ring, M: AddCommGroup](carrier: Module[R, M], n: Nat, v: Fin[n] -> M) {
    fin_vector_smul[R, M](carrier, n, R.1, v) = v
} by {
    let lhs = fin_vector_smul[R, M](carrier, n, R.1, v)
    forall(i: Fin[n]) {
        fin_vector_smul_one_at[R, M](carrier, n, v, i)
        lhs(i) = v(i)
    }
    fin_vector_ext[M](n, lhs, v)
}

/// The zero scalar annihilates a coordinate vector.
theorem fin_vector_smul_zero_left[R: Ring, M: AddCommGroup](carrier: Module[R, M], n: Nat, v: Fin[n] -> M) {
    fin_vector_smul[R, M](carrier, n, R.0, v) = fin_vector_zero[M](n)
} by {
    let lhs = fin_vector_smul[R, M](carrier, n, R.0, v)
    let rhs = fin_vector_zero[M](n)
    forall(i: Fin[n]) {
        fin_vector_smul_zero_left_at[R, M](carrier, n, v, i)
        lhs(i) = rhs(i)
    }
    fin_vector_ext[M](n, lhs, rhs)
}

/// Every scalar annihilates the zero coordinate vector.
theorem fin_vector_smul_zero_right[R: Ring, M: AddCommGroup](carrier: Module[R, M], n: Nat, r: R) {
    fin_vector_smul[R, M](carrier, n, r, fin_vector_zero[M](n)) = fin_vector_zero[M](n)
} by {
    let lhs = fin_vector_smul[R, M](carrier, n, r, fin_vector_zero[M](n))
    let rhs = fin_vector_zero[M](n)
    forall(i: Fin[n]) {
        lhs(i) = carrier.smul(r, fin_vector_zero[M](n, i))
        fin_vector_zero[M](n, i) = M.0
        module_smul_zero_right(carrier, r)
        carrier.smul(r, M.0) = M.0
        rhs(i) = M.0
        lhs(i) = rhs(i)
    }
    fin_vector_ext[M](n, lhs, rhs)
}

/// Scalar multiplication distributes over pointwise negation.
theorem fin_vector_smul_neg[R: Ring, M: AddCommGroup](carrier: Module[R, M], n: Nat, r: R, v: Fin[n] -> M) {
    fin_vector_smul[R, M](carrier, n, r, fin_vector_neg[M](n, v)) =
        fin_vector_neg[M](n, fin_vector_smul[R, M](carrier, n, r, v))
} by {
    let lhs = fin_vector_smul[R, M](carrier, n, r, fin_vector_neg[M](n, v))
    let rhs = fin_vector_neg[M](n, fin_vector_smul[R, M](carrier, n, r, v))
    forall(i: Fin[n]) {
        lhs(i) = carrier.smul(r, fin_vector_neg[M](n, v, i))
        fin_vector_neg[M](n, v, i) = -v(i)
        fin_vector_smul_neg_coord[R, M](carrier, n, r, v, i)
        rhs(i) = -fin_vector_smul[R, M](carrier, n, r, v, i)
        lhs(i) = rhs(i)
    }
    fin_vector_ext[M](n, lhs, rhs)
}
