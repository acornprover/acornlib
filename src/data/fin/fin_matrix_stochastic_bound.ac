from nat import Nat
from data.fin.fin import Fin
from real import Real, lte_trans, lte_abs, abs_gte_zero, mul_abs, only_abs_zero_eq_zero,
    lte_mul_nonneg_left, lt_mul_pos_right, lte_lt_trans, lt_lte_trans
from data.fin.fin_matrix import Matrix, matrix_entry
from data.fin.fin_sum import fin_sum
from data.fin.fin_sum_order import fin_sum_mono
from data.fin.fin_sum_abs import fin_abs, fin_sum_abs_le
from data.fin.fin_sum_scalar import fin_scale, fin_sum_scalar_mul
from data.fin.fin_function_max import fin_function_max, fin_function_max_attained,
    fin_function_max_is_upper_bound
from data.fin.fin_matrix_eigen import matrix_apply, matrix_apply_eq, matrix_apply_term,
    matrix_row_fn, matrix_row_sum, is_eigenvector, is_eigenvector_apply,
    is_eigenvector_nonzero, is_nonzero_vector_witness, is_row_stochastic,
    is_eigenvalue, is_eigenvalue_witness
from data.fin.fin_matrix_stochastic import is_nonnegative_matrix, is_nonnegative_matrix_apply,
    row_stochastic_has_eigenvalue_one

/// A nonnegative real is its own absolute value.
theorem abs_of_not_negative(x: Real) {
    not x.is_negative implies x.abs = x
} by {
    if not x.is_negative {
        x.abs = x
    }
}

/// The largest absolute value taken by a vector is positive when the vector is nonzero.
theorem max_abs_positive(m: Nat, v: Fin[m.suc] -> Real) {
    (exists(j: Fin[m.suc]) { v(j) != Real.0 })
        implies Real.0 < fin_function_max[Real](m, fin_abs(m.suc, v))
} by {
    if exists(j: Fin[m.suc]) { v(j) != Real.0 } {
        let (j: Fin[m.suc]) satisfy {
            v(j) != Real.0
        }
        abs_gte_zero(v(j))
        v(j).abs >= Real.0
        if v(j).abs = Real.0 {
            only_abs_zero_eq_zero(v(j))
            v(j) = Real.0
            false
        }
        v(j).abs != Real.0
        Real.0 < v(j).abs
        fin_function_max_is_upper_bound[Real](m, fin_abs(m.suc, v), j)
        fin_abs(m.suc, v)(j) <= fin_function_max[Real](m, fin_abs(m.suc, v))
        (fin_abs(m.suc, v)(j) = v(j).abs)
        v(j).abs <= fin_function_max[Real](m, fin_abs(m.suc, v))
        lt_lte_trans(Real.0, v(j).abs, fin_function_max[Real](m, fin_abs(m.suc, v)))
        Real.0 < fin_function_max[Real](m, fin_abs(m.suc, v))
    }
}

/// A row of a nonnegative row-stochastic matrix, applied to a vector, is bounded by the largest
/// absolute value the vector takes.
///
/// The termwise bound: each term is at most the row entry times the largest absolute value, and
/// those sum to the row sum times that value, which is the value itself.
theorem stochastic_row_apply_abs_le(
    m: Nat, a: Matrix[Real, m.suc, m.suc], v: Fin[m.suc] -> Real, i: Fin[m.suc]
) {
    is_nonnegative_matrix[Real](m.suc, a) and is_row_stochastic[Real](m.suc, a)
        implies (matrix_apply[Real](m.suc, a, v, i)).abs
            <= fin_function_max[Real](m, fin_abs(m.suc, v))
} by {
    if is_nonnegative_matrix[Real](m.suc, a) and is_row_stochastic[Real](m.suc, a) {
        matrix_apply_eq[Real](m.suc, a, v, i)
        (matrix_apply[Real](m.suc, a, v, i)
            = fin_sum[Real](m.suc, matrix_apply_term[Real](m.suc, a, v, i)))
        fin_sum_abs_le(m.suc, matrix_apply_term[Real](m.suc, a, v, i))
        ((fin_sum[Real](m.suc, matrix_apply_term[Real](m.suc, a, v, i))).abs
            <= fin_sum[Real](m.suc,
                fin_abs(m.suc, matrix_apply_term[Real](m.suc, a, v, i))))
        forall(k: Fin[m.suc]) {
            is_nonnegative_matrix_apply[Real](m.suc, a, i, k)
            Real.0 <= matrix_entry[Real](m.suc, m.suc, a, i, k)
            not matrix_entry[Real](m.suc, m.suc, a, i, k).is_negative
            abs_of_not_negative(matrix_entry[Real](m.suc, m.suc, a, i, k))
            (matrix_entry[Real](m.suc, m.suc, a, i, k).abs
                = matrix_entry[Real](m.suc, m.suc, a, i, k))
            (matrix_apply_term[Real](m.suc, a, v, i)(k)
                = matrix_entry[Real](m.suc, m.suc, a, i, k) * v(k))
            mul_abs(matrix_entry[Real](m.suc, m.suc, a, i, k), v(k))
            ((matrix_entry[Real](m.suc, m.suc, a, i, k) * v(k)).abs
                = matrix_entry[Real](m.suc, m.suc, a, i, k).abs * v(k).abs)
            (fin_abs(m.suc, matrix_apply_term[Real](m.suc, a, v, i))(k)
                = matrix_entry[Real](m.suc, m.suc, a, i, k) * v(k).abs)
            fin_function_max_is_upper_bound[Real](m, fin_abs(m.suc, v), k)
            fin_abs(m.suc, v)(k) <= fin_function_max[Real](m, fin_abs(m.suc, v))
            (fin_abs(m.suc, v)(k) = v(k).abs)
            v(k).abs <= fin_function_max[Real](m, fin_abs(m.suc, v))
            lte_mul_nonneg_left(v(k).abs, fin_function_max[Real](m, fin_abs(m.suc, v)),
                matrix_entry[Real](m.suc, m.suc, a, i, k))
            (matrix_entry[Real](m.suc, m.suc, a, i, k) * v(k).abs
                <= matrix_entry[Real](m.suc, m.suc, a, i, k)
                    * fin_function_max[Real](m, fin_abs(m.suc, v)))
            (fin_scale[Real](m.suc, fin_function_max[Real](m, fin_abs(m.suc, v)),
                matrix_row_fn[Real](m.suc, a, i))(k)
                = fin_function_max[Real](m, fin_abs(m.suc, v))
                    * matrix_row_fn[Real](m.suc, a, i)(k))
            (matrix_row_fn[Real](m.suc, a, i)(k)
                = matrix_entry[Real](m.suc, m.suc, a, i, k))
            (fin_scale[Real](m.suc, fin_function_max[Real](m, fin_abs(m.suc, v)),
                matrix_row_fn[Real](m.suc, a, i))(k)
                = matrix_entry[Real](m.suc, m.suc, a, i, k)
                    * fin_function_max[Real](m, fin_abs(m.suc, v)))
            (fin_abs(m.suc, matrix_apply_term[Real](m.suc, a, v, i))(k)
                <= fin_scale[Real](m.suc, fin_function_max[Real](m, fin_abs(m.suc, v)),
                    matrix_row_fn[Real](m.suc, a, i))(k))
        }
        fin_sum_mono[Real](m.suc, fin_abs(m.suc, matrix_apply_term[Real](m.suc, a, v, i)),
            fin_scale[Real](m.suc, fin_function_max[Real](m, fin_abs(m.suc, v)),
                matrix_row_fn[Real](m.suc, a, i)))
        (fin_sum[Real](m.suc, fin_abs(m.suc, matrix_apply_term[Real](m.suc, a, v, i)))
            <= fin_sum[Real](m.suc,
                fin_scale[Real](m.suc, fin_function_max[Real](m, fin_abs(m.suc, v)),
                    matrix_row_fn[Real](m.suc, a, i))))
        fin_sum_scalar_mul[Real](m.suc, fin_function_max[Real](m, fin_abs(m.suc, v)),
            matrix_row_fn[Real](m.suc, a, i))
        (fin_sum[Real](m.suc,
            fin_scale[Real](m.suc, fin_function_max[Real](m, fin_abs(m.suc, v)),
                matrix_row_fn[Real](m.suc, a, i)))
            = fin_function_max[Real](m, fin_abs(m.suc, v))
                * fin_sum[Real](m.suc, matrix_row_fn[Real](m.suc, a, i)))
        (matrix_row_sum[Real](m.suc, a, i)
            = fin_sum[Real](m.suc, matrix_row_fn[Real](m.suc, a, i)))
        (is_row_stochastic[Real](m.suc, a) = forall(j: Fin[m.suc]) {
            matrix_row_sum[Real](m.suc, a, j) = Real.1
        })
        matrix_row_sum[Real](m.suc, a, i) = Real.1
        fin_sum[Real](m.suc, matrix_row_fn[Real](m.suc, a, i)) = Real.1
        (fin_function_max[Real](m, fin_abs(m.suc, v)) * Real.1
            = fin_function_max[Real](m, fin_abs(m.suc, v)))
        (fin_sum[Real](m.suc, fin_abs(m.suc, matrix_apply_term[Real](m.suc, a, v, i)))
            <= fin_function_max[Real](m, fin_abs(m.suc, v)))
        lte_trans((matrix_apply[Real](m.suc, a, v, i)).abs,
            fin_sum[Real](m.suc, fin_abs(m.suc, matrix_apply_term[Real](m.suc, a, v, i))),
            fin_function_max[Real](m, fin_abs(m.suc, v)))
        ((matrix_apply[Real](m.suc, a, v, i)).abs
            <= fin_function_max[Real](m, fin_abs(m.suc, v)))
    }
}

/// Every eigenvalue of a nonnegative row-stochastic matrix has absolute value at most one.
///
/// Evaluate the eigenvalue equation at an index where the eigenvector attains its largest
/// absolute value. The left side is that value times the absolute value of the eigenvalue; the
/// right side is bounded by the value itself, since the row weights are nonnegative and sum to
/// one. The value is positive because the eigenvector is nonzero, so it cancels.
theorem stochastic_eigenvalue_abs_le_one(
    m: Nat, a: Matrix[Real, m.suc, m.suc], lam: Real, v: Fin[m.suc] -> Real
) {
    is_nonnegative_matrix[Real](m.suc, a) and is_row_stochastic[Real](m.suc, a)
        and is_eigenvector[Real](m.suc, a, lam, v)
        implies lam.abs <= Real.1
} by {
    if is_nonnegative_matrix[Real](m.suc, a) and is_row_stochastic[Real](m.suc, a)
        and is_eigenvector[Real](m.suc, a, lam, v) {
        fin_function_max_attained[Real](m, fin_abs(m.suc, v))
        exists(i: Fin[m.suc]) {
            fin_abs(m.suc, v)(i) = fin_function_max[Real](m, fin_abs(m.suc, v))
        }
        let (i: Fin[m.suc]) satisfy {
            fin_abs(m.suc, v)(i) = fin_function_max[Real](m, fin_abs(m.suc, v))
        }
        (fin_abs(m.suc, v)(i) = v(i).abs)
        v(i).abs = fin_function_max[Real](m, fin_abs(m.suc, v))
        is_eigenvector_nonzero[Real](m.suc, a, lam, v)
        is_nonzero_vector_witness[Real](m.suc, v)
        exists(j: Fin[m.suc]) { v(j) != Real.0 }
        max_abs_positive(m, v)
        Real.0 < fin_function_max[Real](m, fin_abs(m.suc, v))
        is_eigenvector_apply[Real](m.suc, a, lam, v, i)
        matrix_apply[Real](m.suc, a, v, i) = lam * v(i)
        mul_abs(lam, v(i))
        ((lam * v(i)).abs = lam.abs * v(i).abs)
        ((matrix_apply[Real](m.suc, a, v, i)).abs
            = lam.abs * fin_function_max[Real](m, fin_abs(m.suc, v)))
        stochastic_row_apply_abs_le(m, a, v, i)
        ((matrix_apply[Real](m.suc, a, v, i)).abs
            <= fin_function_max[Real](m, fin_abs(m.suc, v)))
        (lam.abs * fin_function_max[Real](m, fin_abs(m.suc, v))
            <= fin_function_max[Real](m, fin_abs(m.suc, v)))
        if Real.1 < lam.abs {
            lt_mul_pos_right(Real.1, lam.abs, fin_function_max[Real](m, fin_abs(m.suc, v)))
            (Real.1 * fin_function_max[Real](m, fin_abs(m.suc, v))
                < lam.abs * fin_function_max[Real](m, fin_abs(m.suc, v)))
            (Real.1 * fin_function_max[Real](m, fin_abs(m.suc, v))
                = fin_function_max[Real](m, fin_abs(m.suc, v)))
            (fin_function_max[Real](m, fin_abs(m.suc, v))
                < lam.abs * fin_function_max[Real](m, fin_abs(m.suc, v)))
            lte_lt_trans(lam.abs * fin_function_max[Real](m, fin_abs(m.suc, v)),
                fin_function_max[Real](m, fin_abs(m.suc, v)),
                lam.abs * fin_function_max[Real](m, fin_abs(m.suc, v)))
            (lam.abs * fin_function_max[Real](m, fin_abs(m.suc, v))
                < lam.abs * fin_function_max[Real](m, fin_abs(m.suc, v)))
            false
        }
        not (Real.1 < lam.abs)
        lam.abs <= Real.1
    }
}

/// One is an eigenvalue of a nonnegative row-stochastic matrix, and no eigenvalue is larger.
///
/// The bound above is attained, so one is an eigenvalue of largest absolute value. The
/// all-ones vector is the eigenvector, which `row_stochastic_has_eigenvalue_one` supplies.
theorem stochastic_spectral_radius_one(m: Nat, a: Matrix[Real, m.suc, m.suc]) {
    is_nonnegative_matrix[Real](m.suc, a) and is_row_stochastic[Real](m.suc, a)
        implies is_eigenvalue[Real](m.suc, a, Real.1) and forall(lam: Real) {
            is_eigenvalue[Real](m.suc, a, lam) implies lam.abs <= Real.1
        }
} by {
    if is_nonnegative_matrix[Real](m.suc, a) and is_row_stochastic[Real](m.suc, a) {
        Real.1 != Real.0
        row_stochastic_has_eigenvalue_one[Real](m, a)
        is_eigenvalue[Real](m.suc, a, Real.1)
        forall(lam: Real) {
            if is_eigenvalue[Real](m.suc, a, lam) {
                is_eigenvalue_witness[Real](m.suc, a, lam)
                exists(v: Fin[m.suc] -> Real) {
                    is_eigenvector[Real](m.suc, a, lam, v)
                }
                let (v: Fin[m.suc] -> Real) satisfy {
                    is_eigenvector[Real](m.suc, a, lam, v)
                }
                stochastic_eigenvalue_abs_le_one(m, a, lam, v)
                lam.abs <= Real.1
            }
            (is_eigenvalue[Real](m.suc, a, lam) implies lam.abs <= Real.1)
        }
        (is_eigenvalue[Real](m.suc, a, Real.1) and forall(mu: Real) {
            is_eigenvalue[Real](m.suc, a, mu) implies mu.abs <= Real.1
        })
    }
}
