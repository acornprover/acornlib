from nat import Nat, lte_and_lt
from data.fin.fin import Fin, fin_new_exists_of_lt, new_round_trip
from list import partial, partial_zero, partial_split_last, partial_pointwise_eq
from real import Real, lte_abs, abs_neg, lte_trans
from data.fin.fin_sum import fin_sum, fin_sum_apply, fin_value_or_zero, fin_value_or_zero_value

/// The triangle inequality for the absolute value.
///
/// `src/real/` exports the pieces — a real is below its absolute value, and negation does not
/// change it — but not the inequality itself, which is what any bound on a sum of terms needs.
theorem abs_add_le(a: Real, b: Real) {
    (a + b).abs <= a.abs + b.abs
} by {
    lte_abs(a)
    a <= a.abs
    lte_abs(b)
    b <= b.abs
    a + b <= a.abs + b.abs
    lte_abs(-a)
    -a <= (-a).abs
    abs_neg(a)
    (-a).abs = a.abs
    -a <= a.abs
    lte_abs(-b)
    -b <= (-b).abs
    abs_neg(b)
    (-b).abs = b.abs
    -b <= b.abs
    -a + -b <= a.abs + b.abs
    (-(a + b) = -a + -b)
    -(a + b) <= a.abs + b.abs
    if (a + b).is_negative {
        (a + b).abs = -(a + b)
        (a + b).abs <= a.abs + b.abs
    }
    if not (a + b).is_negative {
        (a + b).abs = a + b
        (a + b).abs <= a.abs + b.abs
    }
    (a + b).abs <= a.abs + b.abs
}

/// The absolute value of zero is zero.
theorem abs_zero {
    Real.0.abs = Real.0
} by {
    not Real.0.is_negative
    Real.0.abs = Real.0
}

/// The termwise absolute value of a sequence.
define abs_seq(f: Nat -> Real) -> (Nat -> Real) {
    function(k: Nat) {
        f(k).abs
    }
}

/// The absolute value of a partial sum is bounded by the partial sum of absolute values.
///
/// The triangle inequality run along the recurrence.
theorem partial_abs_le(f: Nat -> Real, n: Nat) {
    (partial(f, n)).abs <= partial(abs_seq(f), n)
} by {
    define p(x: Nat) -> Bool {
        (partial(f, x)).abs <= partial(abs_seq(f), x)
    }
    partial_zero(f)
    partial(f, Nat.0) = Real.0
    partial_zero(abs_seq(f))
    partial(abs_seq(f), Nat.0) = Real.0
    abs_zero
    Real.0.abs = Real.0
    Real.0 <= Real.0
    p(Nat.0)
    forall(m: Nat) {
        if p(m) {
            (partial(f, m)).abs <= partial(abs_seq(f), m)
            partial_split_last(f, m)
            partial(f, m.suc) = partial(f, m) + f(m)
            abs_add_le(partial(f, m), f(m))
            ((partial(f, m) + f(m)).abs <= (partial(f, m)).abs + f(m).abs)
            (partial(f, m.suc)).abs <= (partial(f, m)).abs + f(m).abs
            ((partial(f, m)).abs + f(m).abs <= partial(abs_seq(f), m) + f(m).abs)
            lte_trans((partial(f, m.suc)).abs, (partial(f, m)).abs + f(m).abs,
                partial(abs_seq(f), m) + f(m).abs)
            (partial(f, m.suc)).abs <= partial(abs_seq(f), m) + f(m).abs
            (abs_seq(f)(m) = f(m).abs)
            partial_split_last(abs_seq(f), m)
            partial(abs_seq(f), m.suc) = partial(abs_seq(f), m) + abs_seq(f)(m)
            partial(abs_seq(f), m.suc) = partial(abs_seq(f), m) + f(m).abs
            (partial(f, m.suc)).abs <= partial(abs_seq(f), m.suc)
            p(m.suc)
        }
        (p(m) implies p(m.suc))
    }
    p(Nat.0) and forall(m: Nat) {
        p(m) implies p(m.suc)
    }
    Nat.induction(p)
    p(n)
}

/// The termwise absolute value of a function on `Fin[n]`.
define fin_abs(n: Nat, f: Fin[n] -> Real) -> (Fin[n] -> Real) {
    function(i: Fin[n]) {
        f(i).abs
    }
}

/// Zero extension commutes with taking absolute values, below the bound.
///
/// Only below the bound, which is all `partial_pointwise_eq` asks for. Stated through
/// `fin_value_or_zero_value` rather than by matching on the option, since the match branches do
/// not reduce here the way they do inside `src/fin_sum.ac`.
theorem fin_value_or_zero_abs(n: Nat, f: Fin[n] -> Real, k: Nat) {
    k < n implies fin_value_or_zero[Real](n, fin_abs(n, f), k)
        = (fin_value_or_zero[Real](n, f, k)).abs
} by {
    if k < n {
        fin_new_exists_of_lt(n, k)
        exists(j: Fin[n]) {
            Fin[n].new(k) = Option.some(j)
        }
        let (i: Fin[n]) satisfy {
            Fin[n].new(k) = Option.some(i)
        }
        new_round_trip(n, k, i)
        i.value = k
        fin_value_or_zero_value[Real](n, fin_abs(n, f), i)
        fin_value_or_zero[Real](n, fin_abs(n, f), i.value) = fin_abs(n, f)(i)
        fin_value_or_zero[Real](n, fin_abs(n, f), k) = fin_abs(n, f)(i)
        (fin_abs(n, f)(i) = f(i).abs)
        fin_value_or_zero_value[Real](n, f, i)
        fin_value_or_zero[Real](n, f, i.value) = f(i)
        fin_value_or_zero[Real](n, f, k) = f(i)
        (fin_value_or_zero[Real](n, fin_abs(n, f), k)
            = (fin_value_or_zero[Real](n, f, k)).abs)
    }
}

/// The triangle inequality for a finite sum.
///
/// The bound a modulus argument over a finite index set needs: the size of a sum is at most the
/// sum of the sizes.
theorem fin_sum_abs_le(n: Nat, f: Fin[n] -> Real) {
    (fin_sum[Real](n, f)).abs <= fin_sum[Real](n, fin_abs(n, f))
} by {
    fin_sum_apply[Real](n, f)
    fin_sum[Real](n, f) = partial(fin_value_or_zero[Real](n, f), n)
    partial_abs_le(fin_value_or_zero[Real](n, f), n)
    ((partial(fin_value_or_zero[Real](n, f), n)).abs
        <= partial(abs_seq(fin_value_or_zero[Real](n, f)), n))
    forall(k: Nat) {
        if k < n {
            fin_value_or_zero_abs(n, f, k)
            (fin_value_or_zero[Real](n, fin_abs(n, f), k)
                = (fin_value_or_zero[Real](n, f, k)).abs)
            (abs_seq(fin_value_or_zero[Real](n, f))(k)
                = (fin_value_or_zero[Real](n, f, k)).abs)
            (abs_seq(fin_value_or_zero[Real](n, f))(k)
                = fin_value_or_zero[Real](n, fin_abs(n, f), k))
        }
        (k < n implies abs_seq(fin_value_or_zero[Real](n, f))(k)
            = fin_value_or_zero[Real](n, fin_abs(n, f), k))
    }
    partial_pointwise_eq[Real](abs_seq(fin_value_or_zero[Real](n, f)),
        fin_value_or_zero[Real](n, fin_abs(n, f)), n)
    (partial(abs_seq(fin_value_or_zero[Real](n, f)), n)
        = partial(fin_value_or_zero[Real](n, fin_abs(n, f)), n))
    fin_sum_apply[Real](n, fin_abs(n, f))
    (fin_sum[Real](n, fin_abs(n, f))
        = partial(fin_value_or_zero[Real](n, fin_abs(n, f)), n))
    (fin_sum[Real](n, f)).abs <= fin_sum[Real](n, fin_abs(n, f))
}
