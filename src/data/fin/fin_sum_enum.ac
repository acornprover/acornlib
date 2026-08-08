from nat import Nat, lt_not_ref, lt_suc, lt_suc_right
from data.fin.fin import Fin, fin_new_exists_of_lt, fin_of_nat_suc, fin_of_nat_suc_new_of_lt,
    fin_succ, fin_zero, fin_zero_new, new_round_trip, new_self
from data.fin.fin_enum import fin_enum_suc, fin_enum_suc_get_idx, fin_enum_suc_length
from data.fin.fin_sum import fin_sum, fin_value_or_zero
from data.basic.functions import compose
from algebra.zero import Zero
from algebra.add_comm_monoid import AddCommMonoid
from list import List, map, length_range, sum, partial, partial_pointwise_eq, partial_shift_suc,
    partial_split_last, list_extensionality, map_length, map_range,
    map_singleton, sum_add, map_add, map_under_idx

/// Bounded natural indices evaluate the zero-extension at the corresponding
/// successor finite index.
theorem fin_value_or_zero_of_fin_of_nat_suc[A: Zero](n: Nat, f: Fin[n.suc] -> A, k: Nat) {
    k < n.suc implies fin_value_or_zero[A](n.suc, f, k) = f(fin_of_nat_suc(n, k))
} by {
    if k < n.suc {
        fin_of_nat_suc_new_of_lt(n, k)
    }
}

/// The natural-indexed function obtained by applying `f` after constructing a
/// successor finite index.
define fin_of_nat_suc_apply[A](n: Nat, f: Fin[n.suc] -> A, k: Nat) -> A {
    f(fin_of_nat_suc(n, k))
}

/// Mapping over the canonical successor enumeration is the same as mapping the
/// corresponding natural-indexed function over the natural range.
theorem map_fin_enum_suc_eq_range_apply[A](n: Nat, f: Fin[n.suc] -> A) {
    map[Fin[n.suc], A](fin_enum_suc(n), f) = map[Nat, A](n.suc.range, fin_of_nat_suc_apply[A](n, f))
} by {
    let left = map[Fin[n.suc], A](fin_enum_suc(n), f)
    let right = map[Nat, A](n.suc.range, fin_of_nat_suc_apply[A](n, f))
    fin_enum_suc_length(n)
    map_length[Fin[n.suc], A](fin_enum_suc(n), f)
    map_length[Nat, A](n.suc.range, fin_of_nat_suc_apply[A](n, f))
    length_range(n.suc)
    left.length = right.length

    forall(idx: Nat) {
        if idx < left.length {
            idx < n.suc
            fin_enum_suc_get_idx(n, idx)
            map_under_idx[Fin[n.suc], A](fin_enum_suc(n), f, idx)
            exists(x: Fin[n.suc]) {
                Option.some(x) = fin_enum_suc(n).get_idx(idx) and
                map[Fin[n.suc], A](fin_enum_suc(n), f).get_idx(idx) = Option.some(f(x))
            }
            let x: Fin[n.suc] satisfy {
                Option.some(x) = fin_enum_suc(n).get_idx(idx) and
                left.get_idx(idx) = Option.some(f(x))
            }
            some_injective[Fin[n.suc]](x, fin_of_nat_suc(n, idx))
            x = fin_of_nat_suc(n, idx)
            left.get_idx(idx) = Option.some(f(fin_of_nat_suc(n, idx)))
            map_range[A](n.suc, idx, fin_of_nat_suc_apply[A](n, f))
            right.get_idx(idx) = Option.some(fin_of_nat_suc_apply[A](n, f, idx))
            fin_of_nat_suc_apply[A](n, f, idx) = f(fin_of_nat_suc(n, idx))
            right.get_idx(idx) = Option.some(f(fin_of_nat_suc(n, idx)))
            left.get_idx(idx) = right.get_idx(idx)
        }
    }
    left.length = right.length and (forall(idx: Nat) { idx < left.length implies left.get_idx(idx) = right.get_idx(idx) })
    list_extensionality[A](left, right)
    left = right
}

/// The finite sum over `Fin[n + 1]` agrees with the list sum over the canonical
/// successor enumeration.
theorem fin_sum_suc_eq_sum_fin_enum_suc[A: AddCommMonoid](n: Nat, f: Fin[n.suc] -> A) {
    fin_sum[A](n.suc, f) = sum(map(fin_enum_suc(n), f))
} by {
    let value = fin_value_or_zero[A](n.suc, f)
    let enum_fun = fin_of_nat_suc_apply[A](n, f)
    forall(k: Nat) {
        if k < n.suc {
            fin_value_or_zero_of_fin_of_nat_suc[A](n, f, k)
            fin_of_nat_suc_apply[A](n, f, k) = f(fin_of_nat_suc(n, k))
            value(k) = f(fin_of_nat_suc(n, k))
            enum_fun(k) = f(fin_of_nat_suc(n, k))
            value(k) = enum_fun(k)
        }
    }
    partial_pointwise_eq[A](value, enum_fun, n.suc)
    fin_sum[A](n.suc, f) = partial[A](value, n.suc)
    partial[A](value, n.suc) = partial[A](enum_fun, n.suc)
    partial[A](enum_fun, n.suc) = sum[A](map[Nat, A](n.suc.range, enum_fun))
    map_fin_enum_suc_eq_range_apply[A](n, f)
    sum[A](map[Fin[n.suc], A](fin_enum_suc(n), f)) = sum[A](map[Nat, A](n.suc.range, enum_fun))
    fin_sum[A](n.suc, f) = sum[A](map[Fin[n.suc], A](fin_enum_suc(n), f))
}

/// A natural-indexed function with one nonzero value.
define nat_single_nonzero[A: Zero](value: A, target: Nat, k: Nat) -> A {
    if k = target {
        value
    } else {
        A.0
    }
}

/// Before the selected natural index, the single-nonzero partial sum is zero.
theorem partial_nat_single_nonzero_zero_before[A: AddCommMonoid](value: A, target: Nat, n: Nat) {
    n <= target implies partial[A](nat_single_nonzero[A](value, target), n) = A.0
} by {
    let h = nat_single_nonzero[A](value, target)
    define p(k: Nat) -> Bool {
        k <= target implies partial[A](h, k) = A.0
    }
    p(Nat.0)
    forall(k: Nat) {
        if p(k) {
            if k.suc <= target {
                k <= target
                p(k)
                partial[A](h, k) = A.0
                partial_split_last[A](h, k)
                partial[A](h, k.suc) = partial[A](h, k) + h(k)
                if k = target {
                    k.suc <= k
                    lt_suc(k)
                    k < k.suc
                    k < k
                    lt_not_ref(k)
                    false
                }
                k != target
                h(k) = A.0
                partial[A](h, k.suc) = A.0 + A.0
                A.0 + A.0 = A.0
                partial[A](h, k.suc) = A.0
            }
            p(k.suc)
        }
    }
    p(Nat.0) and forall(k: Nat) { p(k) implies p(k.suc) }
    p(n)
}

/// Once the selected natural index is included, the single-nonzero partial sum is that value.
theorem partial_nat_single_nonzero_of_lt[A: AddCommMonoid](value: A, target: Nat, n: Nat) {
    target < n implies partial[A](nat_single_nonzero[A](value, target), n) = value
} by {
    let h = nat_single_nonzero[A](value, target)
    define p(k: Nat) -> Bool {
        target < k implies partial[A](h, k) = value
    }
    p(Nat.0)
    forall(k: Nat) {
        if p(k) {
            if target < k.suc {
                partial_split_last[A](h, k)
                partial[A](h, k.suc) = partial[A](h, k) + h(k)
                if target = k {
                    k = target
                    k <= target
                    partial_nat_single_nonzero_zero_before[A](value, target, k)
                    partial[A](h, k) = A.0
                    h(k) = value
                    partial[A](h, k.suc) = A.0 + value
                    A.0 + value = value
                    partial[A](h, k.suc) = value
                } else {
                    lt_suc_right(target, k)
                    target < k
                    p(k)
                    partial[A](h, k) = value
                    k != target
                    h(k) = A.0
                    partial[A](h, k.suc) = value + A.0
                    value + A.0 = value
                    partial[A](h, k.suc) = value
                }
            }
            p(k.suc)
        }
    }
    p(Nat.0) and forall(k: Nat) { p(k) implies p(k.suc) }
    p(n)
}

/// A finite sum with a single possible nonzero index is that value.
theorem fin_sum_single_nonzero[A: AddCommMonoid](n: Nat, f: Fin[n] -> A, target: Fin[n]) {
    (forall(i: Fin[n]) { i != target implies f(i) = A.0 }) implies
        fin_sum[A](n, f) = f(target)
} by {
    if forall(i: Fin[n]) { i != target implies f(i) = A.0 } {
        let value = fin_value_or_zero[A](n, f)
        let h = nat_single_nonzero[A](f(target), target.value)
        forall(k: Nat) {
            if k < n {
                if k = target.value {
                    new_self(n, target)
                    Fin[n].new(k) = Option.some(target)
                    value(k) = f(target)
                    h(k) = f(target)
                    value(k) = h(k)
                } else {
                    fin_new_exists_of_lt(n, k)
                    let i: Fin[n] satisfy {
                        Fin[n].new(k) = Option.some(i)
                    }
                    new_round_trip(n, k, i)
                    i.value = k
                    if i = target {
                        target.value = k
                        false
                    }
                    i != target
                    f(i) = A.0
                    value(k) = f(i)
                    h(k) = A.0
                    value(k) = h(k)
                }
            }
        }
        partial_pointwise_eq[A](value, h, n)
        fin_sum[A](n, f) = partial[A](value, n)
        partial[A](value, n) = partial[A](h, n)
        target.value < n
        partial_nat_single_nonzero_of_lt[A](f(target), target.value, n)
        partial[A](h, n) = f(target)
        fin_sum[A](n, f) = f(target)
    }
}

/// Summing over the canonical successor enumeration agrees with the finite sum.
theorem sum_fin_enum_suc_eq_fin_sum[A: AddCommMonoid](n: Nat, f: Fin[n.suc] -> A) {
    sum(map(fin_enum_suc(n), f)) = fin_sum[A](n.suc, f)
} by {
    fin_sum_suc_eq_sum_fin_enum_suc[A](n, f)
}

/// Mapping a function over the singleton list containing a finite index produces the singleton value list.
theorem map_fin_singleton[A](n: Nat, f: Fin[n] -> A, i: Fin[n]) {
    map(List.singleton(i), f) = List.singleton(f(i))
} by {
    map_singleton[Fin[n], A](f, i)
}

/// The list sum over a singleton finite-index list is the value at that index.
theorem sum_map_fin_singleton[A: AddCommMonoid](n: Nat, f: Fin[n] -> A, i: Fin[n]) {
    sum(map(List.singleton(i), f)) = f(i)
} by {
    map_fin_singleton[A](n, f, i)
    sum(map(List.singleton(i), f)) = sum(List.singleton(f(i)))
    sum(List.singleton(f(i))) = f(i)
}

/// List-summing mapped finite-index functions distributes over append.
theorem sum_map_fin_append[A: AddCommMonoid](n: Nat, left: List[Fin[n]], right: List[Fin[n]], f: Fin[n] -> A) {
    sum(map(left + right, f)) = sum(map(left, f)) + sum(map(right, f))
} by {
    map_add[Fin[n], A](left, right, f)
    sum_add[A](map(left, f), map(right, f))
}

/// Finite sums over `Fin[n + 1]` can be evaluated through the enumeration map.
theorem fin_sum_suc_map_enum_bridge[A: AddCommMonoid](n: Nat, f: Fin[n.suc] -> A) {
    fin_sum[A](n.suc, f) = sum(map(fin_enum_suc(n), f)) and
    sum(map(fin_enum_suc(n), f)) = fin_sum[A](n.suc, f)
} by {
    fin_sum_suc_eq_sum_fin_enum_suc[A](n, f)
    sum_fin_enum_suc_eq_fin_sum[A](n, f)
}

/// The shifted function obtained by applying `f` after the successor inclusion
/// `Fin[n] -> Fin[n + 1]`.
define fin_succ_apply[A](n: Nat, f: Fin[n.suc] -> A, i: Fin[n]) -> A {
    f(fin_succ(n, i))
}

/// On a bounded tail index, the zero-extensions for a successor-bound function
/// and its successor-reindexed tail agree.
theorem fin_value_or_zero_succ_index[A: Zero](n: Nat, f: Fin[n.suc] -> A, k: Nat) {
    k < n implies fin_value_or_zero[A](n.suc, f, k.suc) =
        fin_value_or_zero[A](n, fin_succ_apply[A](n, f), k)
} by {
    if k < n {
        fin_new_exists_of_lt(n, k)
        let i: Fin[n] satisfy {
            Fin[n].new(k) = Option.some(i)
        }
        new_round_trip(n, k, i)
        i.value = k
        Fin[n.suc].new(i.value.suc) = Option.some(fin_succ(n, i))
        Fin[n.suc].new(k.suc) = Option.some(fin_succ(n, i))
        fin_value_or_zero[A](n.suc, f, k.suc) = f(fin_succ(n, i))
        fin_value_or_zero[A](n, fin_succ_apply[A](n, f), k) = fin_succ_apply[A](n, f, i)
        fin_succ_apply[A](n, f, i) = f(fin_succ(n, i))
        fin_value_or_zero[A](n.suc, f, k.suc) =
            fin_value_or_zero[A](n, fin_succ_apply[A](n, f), k)
    }
}

/// Splitting the successor finite sum at zero: the head term is the first
/// finite index and the tail is reindexed through `fin_succ`.
theorem fin_sum_suc_split_zero[A: AddCommMonoid](n: Nat, f: Fin[n.suc] -> A) {
    fin_sum[A](n.suc, f) = f(fin_zero(n)) + fin_sum[A](n, fin_succ_apply[A](n, f))
} by {
    let value = fin_value_or_zero[A](n.suc, f)
    let tail_value = fin_value_or_zero[A](n, fin_succ_apply[A](n, f))
    forall(k: Nat) {
        if k < n {
            compose[Nat, Nat, A](value, Nat.suc, k) = value(k.suc)
            fin_value_or_zero_succ_index[A](n, f, k)
            value(k.suc) = tail_value(k)
            compose[Nat, Nat, A](value, Nat.suc, k) = tail_value(k)
        }
    }
    partial_pointwise_eq[A](compose[Nat, Nat, A](value, Nat.suc), tail_value, n)
    partial[A](compose[Nat, Nat, A](value, Nat.suc), n) = partial[A](tail_value, n)
    fin_sum[A](n, fin_succ_apply[A](n, f)) = partial[A](tail_value, n)

    fin_value_or_zero_of_fin_of_nat_suc[A](n, f, Nat.0)
    fin_of_nat_suc_new_of_lt(n, Nat.0)
    fin_zero_new(n)
    Option.some(fin_of_nat_suc(n, Nat.0)) = Option.some(fin_zero(n))
    some_injective[Fin[n.suc]](fin_of_nat_suc(n, Nat.0), fin_zero(n))
    fin_of_nat_suc(n, Nat.0) = fin_zero(n)
    value(Nat.0) = f(fin_zero(n))

    partial_shift_suc[A](value, n)
    value(Nat.0) + partial[A](compose[Nat, Nat, A](value, Nat.suc), n) = partial[A](value, n.suc)
    f(fin_zero(n)) + fin_sum[A](n, fin_succ_apply[A](n, f)) = partial[A](value, n.suc)
    fin_sum[A](n.suc, f) = partial[A](value, n.suc)
    fin_sum[A](n.suc, f) = f(fin_zero(n)) + fin_sum[A](n, fin_succ_apply[A](n, f))
}
